/// Classical inequalities: Young, Hölder, Minkowski, and the
/// arithmetic-geometric-harmonic mean chain.
///
/// Young's inequality (the weighted AM-GM inequality for two values) is
/// derived from the lower tangent-line bound x.exp >= 1 + x of the
/// exponential function.  Hölder's inequality is the finite-sequence form
/// obtained by summing Young's inequality over normalized sequences;
/// Minkowski's inequality follows from Hölder's.  The harmonic mean is
/// bounded by the geometric mean by applying AM-GM to the sequence of
/// reciprocals.

from nat import Nat, from_nat, lt_and_lte, zero_or_suc, alt_induction, lt_suc_right, not_lt_zero
from list import partial, partial_add, partial_split_last, partial_zero, partial_pointwise_eq, partial_scalar_mul
from order import lte_antisymm, lte_trans, lt_trans, lt_imp_lte, lt_of_lt_of_lte, lt_of_lte_of_lt, not_gt_imp_lte, not_lte_imp_gt, not_lt_imp_gte, lt_irrefl, lte_refl, lte_lt_trans
from algebra.add_semigroup import add_fn
from algebra.semigroup import mul_fn
from real.real_base import Real, abs_gte_zero, lte_add_right, lte_add_left, lt_add_right, lt_add_left, add_lte_add, add_neg_eq_zero, pos_gt_zero, gt_zero_imp_pos, neg_neg, add_comm, add_assoc, add_zero_left, add_zero_right, sub_cancels, neg_distrib, neg_zero, lt_add_pos, neg_lt_zero, neg_lt_swap_neg
from real.real_ring import mul_abs, mul_nonneg, mul_pos_pos, mul_neg_left, mul_neg_right, lte_mul_nonneg_left, lte_mul_nonneg_right, mul_le_mul_nonneg, real_mul_comm, mul_assoc, mul_distrib_right, mul_distrib_left, mul_one_left, mul_one_right, mul_zero_left, mul_zero_right, square_nonneg, lt_mul_pos_right
from real.real_field import mul_div_cancel, mul_inverse, div_le_of_mul_le, mul_le_mul_pos_right, mul_le_mul_pos_left, mul_div, div_div, div_by_fraction
from real.harmonic import real_one_div_pos, real_recip_antitone_pos, real_inverse_antitone_pos_strict, from_nat_suc_pos_real
from real.exp import exp_add, exp_pos, exp_zero, exp_increasing, pow_suc, pow_pos, zero_pow_pos, mul_frac_assoc, mul_one_over, mul_assoc_real, div_lt_div_pos
from real.exp_inequalities import exp_monotone, exp_ge_one_plus_self, one_div_one_div, self_le_exp_sub_one
from real.log import log_some_of_pos_exists, exp_log_or_zero, log_exp, exp_neg, exp_injective, rpow_zero_base_pos, rpow_one, rpow_add, rpow_mul, rpow_nat, rpow_pos, rpow_zero
from real.log_exp_foundations import log_value_one, log_value_e, log_value_exp, exp_sub, log_value_mul, log_value_div, log_value_recip, log_value_rpow, log_rpow_val
from real.log_inequalities import log_monotone
from real.rpow_deep import rpow_prod_base, rpow_prod_base_val, rpow_quot_base, rpow_quot_base_val, rpow_inv_base, rpow_rpow, rpow_mul_val, rpow_neg, rpow_sub, rpow_of_one
from real.real_series import is_lower_bound, partial_nonneg, triangle_ineq, partial_all_zeros
from real.real_seq import only_abs_zero_eq_zero
from real.convex import convex_on, convex_on_apply, derivative_nondecreasing_imp_convex
from real.calculus_api import is_derivative_fn
from real.derivative_exp_log import exp_is_derivative_fn
from real.finite_product_mean import finite_real_product, finite_real_mean, nonnegative_on, finite_real_product_suc, finite_real_product_zero, nonnegative_on_prefix, finite_real_product_nonnegative, from_nat_real_pos_of_ne_zero, from_nat_real_ne_zero_of_ne_zero, finite_real_mean_mul_count
from real.am_gm import positive_on, positive_on_prefix, positive_on_imp_nonnegative_on, finite_real_product_pos, partial_nonneg_bounded, add_nonneg_pos, partial_pos_bounded, div_pos_of_pos_pos, div_nonneg_of_nonneg_pos, partial_le_range, partial_const, partial_sub_const, partial_div_const, partial_neg, const_real_fn, sub_const_fn, div_const_fn, neg_fn, partial_nonneg_eq_zero_imp_each, finite_real_product_const, add_nonneg_eq_zero_left, add_nonneg_eq_zero_right, am_gm, am_gm_pos, am_gm_zero

numerals Nat
numerals Real

// =====================================================================
// Real powers of nonnegative bases
// =====================================================================

/// The value of the real power of a nonnegative base with a positive
/// exponent: the exponential form for positive bases and zero for the zero
/// base.
define rpow_root_value(base: Real, exponent: Real) -> Real {
    if base > Real.0 {
        (exponent * base.log.get_or_else(Real.0)).exp
    } else {
        Real.0
    }
}

/// A nonnegative base to a positive exponent takes the exponential-form value.
theorem rpow_root_value_eq(base: Real, exponent: Real) {
    base >= Real.0 and exponent > Real.0 implies
        base.rpow(exponent) = Option.some(rpow_root_value(base, exponent))
} by {
    if base >= Real.0 and exponent > Real.0 {
        if base > Real.0 {
            base.rpow(exponent) = Option.some((exponent * base.log.get_or_else(Real.0)).exp)
            rpow_root_value(base, exponent) = (exponent * base.log.get_or_else(Real.0)).exp
            base.rpow(exponent) = Option.some(rpow_root_value(base, exponent))
        } else {
            not base > Real.0
            not_gt_imp_lte(base, Real.0)
            base <= Real.0
            base >= Real.0
            lte_antisymm(base, Real.0)
            base = Real.0
            rpow_zero_base_pos(exponent)
            (Real.0).rpow(exponent) = Option.some(Real.0)
            rpow_root_value(base, exponent) = Real.0
            base.rpow(exponent) = Option.some(rpow_root_value(base, exponent))
        }
    }
}

/// Every nonnegative base to a positive exponent has a nonnegative power value.
theorem rpow_nonneg_base_value(base: Real, exponent: Real) {
    base >= Real.0 and exponent > Real.0 implies exists(value: Real) {
        base.rpow(exponent) = Option.some(value) and value >= Real.0
    }
} by {
    if base >= Real.0 and exponent > Real.0 {
        if base > Real.0 {
            rpow_pos(base, exponent)
            let (v: Real) satisfy {
                base.rpow(exponent) = Option.some(v) and v > Real.0
            }
            v > Real.0
            v >= Real.0
            exists(value: Real) {
                base.rpow(exponent) = Option.some(value) and value >= Real.0
            }
        } else {
            not base > Real.0
            not_gt_imp_lte(base, Real.0)
            base <= Real.0
            base >= Real.0
            lte_antisymm(base, Real.0)
            base = Real.0
            rpow_zero_base_pos(exponent)
            (Real.0).rpow(exponent) = Option.some(Real.0)
            base.rpow(exponent) = Option.some(Real.0)
            Real.0 >= Real.0
            exists(value: Real) {
                base.rpow(exponent) = Option.some(value) and value >= Real.0
            }
        }
    }
}

// =====================================================================
// The p-th power of absolute values of a sequence
// =====================================================================

/// The p-th power of the absolute value of a sequence entry.
define abs_pow_value(a: Nat -> Real, p: Real, i: Nat) -> Real {
    if a(i).abs > Real.0 {
        (p * (a(i).abs).log.get_or_else(Real.0)).exp
    } else {
        Real.0
    }
}

/// The product of the absolute values of two sequence entries.
define abs_product(a: Nat -> Real, b: Nat -> Real, i: Nat) -> Real {
    (a(i) * b(i)).abs
}

/// The sum of the p-th powers of the absolute values of the first `n` entries.
define abs_pow_sum(a: Nat -> Real, p: Real, n: Nat) -> Real {
    partial(abs_pow_value(a, p), n)
}

/// The p-th power of an absolute value is nonnegative for positive p.
theorem abs_pow_value_nonneg(a: Nat -> Real, p: Real, i: Nat) {
    p > Real.0 implies abs_pow_value(a, p, i) >= Real.0
} by {
    if p > Real.0 {
        if a(i).abs > Real.0 {
            exp_pos(p * (a(i).abs).log.get_or_else(Real.0))
            (p * (a(i).abs).log.get_or_else(Real.0)).exp > Real.0
            abs_pow_value(a, p, i) = (p * (a(i).abs).log.get_or_else(Real.0)).exp
            abs_pow_value(a, p, i) > Real.0
            abs_pow_value(a, p, i) >= Real.0
        } else {
            abs_pow_value(a, p, i) = Real.0
            abs_pow_value(a, p, i) >= Real.0
        }
    }
}

/// The real power of an absolute value at index `i` has value `abs_pow_value`.
theorem abs_pow_value_rpow(a: Nat -> Real, p: Real, i: Nat) {
    p > Real.0 implies (a(i).abs).rpow(p) = Option.some(abs_pow_value(a, p, i))
} by {
    if p > Real.0 {
        abs_gte_zero(a(i))
        a(i).abs >= Real.0
        rpow_root_value_eq(a(i).abs, p)
        (a(i).abs).rpow(p) = Option.some(rpow_root_value(a(i).abs, p))
        if a(i).abs > Real.0 {
            abs_pow_value(a, p, i) = (p * (a(i).abs).log.get_or_else(Real.0)).exp
            rpow_root_value(a(i).abs, p) = (p * (a(i).abs).log.get_or_else(Real.0)).exp
            rpow_root_value(a(i).abs, p) = abs_pow_value(a, p, i)
            (a(i).abs).rpow(p) = Option.some(abs_pow_value(a, p, i))
        } else {
            abs_pow_value(a, p, i) = Real.0
            rpow_root_value(a(i).abs, p) = Real.0
            rpow_root_value(a(i).abs, p) = abs_pow_value(a, p, i)
            (a(i).abs).rpow(p) = Option.some(abs_pow_value(a, p, i))
        }
    }
}

/// The p-th power of a zero absolute value is zero.
theorem abs_pow_value_of_abs_zero(a: Nat -> Real, p: Real, i: Nat) {
    a(i).abs = Real.0 implies abs_pow_value(a, p, i) = Real.0
} by {
    if a(i).abs = Real.0 {
        not a(i).abs > Real.0
        abs_pow_value(a, p, i) = Real.0
    }
}

/// A zero p-th power of an absolute value forces the absolute value to be zero.
theorem abs_pow_value_zero_imp_abs_zero(a: Nat -> Real, p: Real, i: Nat) {
    p > Real.0 and abs_pow_value(a, p, i) = Real.0 implies a(i).abs = Real.0
} by {
    if p > Real.0 and abs_pow_value(a, p, i) = Real.0 {
        if a(i).abs > Real.0 {
            abs_pow_value(a, p, i) = (p * (a(i).abs).log.get_or_else(Real.0)).exp
            exp_pos(p * (a(i).abs).log.get_or_else(Real.0))
            (p * (a(i).abs).log.get_or_else(Real.0)).exp > Real.0
            abs_pow_value(a, p, i) > Real.0
            abs_pow_value(a, p, i) != Real.0
            abs_pow_value(a, p, i) = Real.0
            false
        }
        not a(i).abs > Real.0
        not_gt_imp_lte(a(i).abs, Real.0)
        a(i).abs <= Real.0
        abs_gte_zero(a(i))
        a(i).abs >= Real.0
        lte_antisymm(a(i).abs, Real.0)
        a(i).abs = Real.0
    }
}

/// The 1/p-th power of the p-th power of an absolute value is the absolute value.
theorem abs_pow_value_root(a: Nat -> Real, p: Real, i: Nat) {
    p > Real.0 implies
        (abs_pow_value(a, p, i)).rpow(Real.1 / p) = Option.some(a(i).abs)
} by {
    if p > Real.0 {
        if a(i).abs > Real.0 {
            abs_pow_value(a, p, i) = (p * (a(i).abs).log.get_or_else(Real.0)).exp
            log_value_exp(p * (a(i).abs).log.get_or_else(Real.0))
            ((p * (a(i).abs).log.get_or_else(Real.0)).exp).log.get_or_else(Real.0) = p * (a(i).abs).log.get_or_else(Real.0)
            (abs_pow_value(a, p, i)).log.get_or_else(Real.0) = ((p * (a(i).abs).log.get_or_else(Real.0)).exp).log.get_or_else(Real.0)
            (abs_pow_value(a, p, i)).log.get_or_else(Real.0) = p * (a(i).abs).log.get_or_else(Real.0)
            exp_pos(p * (a(i).abs).log.get_or_else(Real.0))
            (p * (a(i).abs).log.get_or_else(Real.0)).exp > Real.0
            abs_pow_value(a, p, i) > Real.0
            (abs_pow_value(a, p, i)).rpow(Real.1 / p) = Option.some(((Real.1 / p) * (abs_pow_value(a, p, i)).log.get_or_else(Real.0)).exp)
            (abs_pow_value(a, p, i)).log.get_or_else(Real.0) = p * (a(i).abs).log.get_or_else(Real.0)
            (Real.1 / p) * (abs_pow_value(a, p, i)).log.get_or_else(Real.0) = (Real.1 / p) * (p * (a(i).abs).log.get_or_else(Real.0))
            (Real.1 / p) * (p * (a(i).abs).log.get_or_else(Real.0)) = ((Real.1 / p) * p) * (a(i).abs).log.get_or_else(Real.0)
            p != Real.0
            mul_div_cancel(Real.1, p)
            p * (Real.1 / p) = Real.1
            (Real.1 / p) * p = Real.1
            mul_one_left((a(i).abs).log.get_or_else(Real.0))
            Real.1 * (a(i).abs).log.get_or_else(Real.0) = (a(i).abs).log.get_or_else(Real.0)
            ((Real.1 / p) * p) * (a(i).abs).log.get_or_else(Real.0) = (a(i).abs).log.get_or_else(Real.0)
            (Real.1 / p) * (abs_pow_value(a, p, i)).log.get_or_else(Real.0) = (a(i).abs).log.get_or_else(Real.0)
            ((Real.1 / p) * (abs_pow_value(a, p, i)).log.get_or_else(Real.0)).exp = ((a(i).abs).log.get_or_else(Real.0)).exp
            log_some_of_pos_exists(a(i).abs)
            let la: Real satisfy {
                a(i).abs.log = Option.some(la)
            }
            (a(i).abs).log.get_or_else(Real.0) = la
            exp_log_or_zero(a(i).abs, la)
            la.exp = a(i).abs
            ((a(i).abs).log.get_or_else(Real.0)).exp = a(i).abs
            ((Real.1 / p) * (abs_pow_value(a, p, i)).log.get_or_else(Real.0)).exp = a(i).abs
            Option.some(((Real.1 / p) * (abs_pow_value(a, p, i)).log.get_or_else(Real.0)).exp) = Option.some(a(i).abs)
            (abs_pow_value(a, p, i)).rpow(Real.1 / p) = Option.some(a(i).abs)
        } else {
            not a(i).abs > Real.0
            not_gt_imp_lte(a(i).abs, Real.0)
            a(i).abs <= Real.0
            abs_gte_zero(a(i))
            a(i).abs >= Real.0
            lte_antisymm(a(i).abs, Real.0)
            a(i).abs = Real.0
            abs_pow_value_of_abs_zero(a, p, i)
            abs_pow_value(a, p, i) = Real.0
            p > Real.0
            real_one_div_pos(p)
            Real.1 / p > Real.0
            rpow_zero_base_pos(Real.1 / p)
            (Real.0).rpow(Real.1 / p) = Option.some(Real.0)
            (abs_pow_value(a, p, i)).rpow(Real.1 / p) = Option.some(Real.0)
            a(i).abs = Real.0
            (abs_pow_value(a, p, i)).rpow(Real.1 / p) = Option.some(a(i).abs)
        }
    }
}

/// The 1/p-th power of a normalized p-th power of an absolute value is the
/// absolute value divided by the p-norm.
theorem normalized_abs_pow_root_eq(a: Nat -> Real, p: Real, aa: Real, uu: Real, i: Nat) {
    p > Real.1 and aa > Real.0 and aa.rpow(Real.1 / p) = Option.some(uu)
    implies (abs_pow_value(a, p, i) / aa).rpow(Real.1 / p) = Option.some(a(i).abs / uu)
} by {
    if p > Real.1 and aa > Real.0 and aa.rpow(Real.1 / p) = Option.some(uu) {
        if a(i).abs > Real.0 {
            abs_pow_value(a, p, i) = (p * (a(i).abs).log.get_or_else(Real.0)).exp
            exp_pos(p * (a(i).abs).log.get_or_else(Real.0))
            (p * (a(i).abs).log.get_or_else(Real.0)).exp > Real.0
            abs_pow_value(a, p, i) > Real.0
            Real.0 < Real.1
            lt_trans(Real.0, Real.1, p)
            Real.0 < p
            p > Real.0
            abs_pow_value_root(a, p, i)
            (abs_pow_value(a, p, i)).rpow(Real.1 / p) = Option.some(a(i).abs)
            rpow_quot_base_val(abs_pow_value(a, p, i), aa, Real.1 / p, a(i).abs, uu)
            (abs_pow_value(a, p, i) / aa).rpow(Real.1 / p) = Option.some(a(i).abs / uu)
        } else {
            not a(i).abs > Real.0
            not_gt_imp_lte(a(i).abs, Real.0)
            a(i).abs <= Real.0
            abs_gte_zero(a(i))
            a(i).abs >= Real.0
            lte_antisymm(a(i).abs, Real.0)
            a(i).abs = Real.0
            abs_pow_value_of_abs_zero(a, p, i)
            abs_pow_value(a, p, i) = Real.0
            aa != Real.0
            abs_pow_value(a, p, i) / aa = Real.0
            Real.0 < Real.1
            lt_trans(Real.0, Real.1, p)
            Real.0 < p
            p > Real.0
            real_one_div_pos(p)
            Real.1 / p > Real.0
            rpow_zero_base_pos(Real.1 / p)
            (Real.0).rpow(Real.1 / p) = Option.some(Real.0)
            (abs_pow_value(a, p, i) / aa).rpow(Real.1 / p) = Option.some(Real.0)
            a(i).abs / uu = Real.0
            (abs_pow_value(a, p, i) / aa).rpow(Real.1 / p) = Option.some(a(i).abs / uu)
        }
    }
}

// =====================================================================
// Young's inequality (weighted AM-GM for two values)
// =====================================================================

/// The reciprocal of an exponent above one is positive.
theorem recip_pos_of_gt_one(p: Real) {
    p > Real.1 implies Real.0 < Real.1 / p
} by {
    if p > Real.1 {
        Real.0 < Real.1
        lt_trans(Real.0, Real.1, p)
        Real.0 < p
        p > Real.0
        real_one_div_pos(p)
        Real.1 / p > Real.0
        Real.0 < Real.1 / p
    }
}

/// The reciprocal of an exponent above one is below one.
theorem recip_lt_one_of_gt_one(p: Real) {
    p > Real.1 implies Real.1 / p < Real.1
} by {
    if p > Real.1 {
        Real.0 < Real.1
        lt_trans(Real.0, Real.1, p)
        Real.0 < p
        p > Real.0
        real_one_div_pos(p)
        Real.1 / p > Real.0
        div_lt_div_pos(Real.1, p, p)
        Real.1 / p < p / p
        mul_div_cancel(Real.1, p)
        p * (Real.1 / p) = Real.1
        p / p = Real.1
        Real.1 / p < Real.1
    }
}

/// An exponent above one is positive.
theorem gt_one_imp_pos(x: Real) {
    x > Real.1 implies x > Real.0
} by {
    if x > Real.1 {
        Real.0 < Real.1
        lt_trans(Real.0, Real.1, x)
        Real.0 < x
        x > Real.0
    }
}

/// Conjugate exponents satisfy 1 - 1/p = 1/q.
theorem conjugate_recip(p: Real, q: Real) {
    Real.1 / p + Real.1 / q = Real.1 implies Real.1 - Real.1 / p = Real.1 / q
} by {
    if Real.1 / p + Real.1 / q = Real.1 {
        Real.1 - Real.1 / p = Real.1 / q
    }
}

/// The exponential function is convex on every interval.
theorem exp_convex_on(a: Real, b: Real) {
    a < b implies convex_on(Real.exp, a, b)
} by {
    if a < b {
        exp_is_derivative_fn
        is_derivative_fn(Real.exp, Real.exp)
        forall(u: Real, v: Real) {
            if u <= v {
                exp_monotone(u, v)
                u.exp <= v.exp
            }
        }
        derivative_nondecreasing_imp_convex(Real.exp, Real.exp, a, b)
        convex_on(Real.exp, a, b)
    }
}

/// The exponential function satisfies the weighted AM-GM inequality: for
/// conjugate exponents p, q > 1 and positive x, y,
/// (log x / p + log y / q).exp <= x / p + y / q.
theorem exp_convex_combine(x: Real, y: Real, p: Real, q: Real) {
    x > Real.0 and y > Real.0 and p > Real.1 and q > Real.1 and Real.1 / p + Real.1 / q = Real.1
    implies (x.log.get_or_else(Real.0) / p + y.log.get_or_else(Real.0) / q).exp <= x / p + y / q
} by {
    if x > Real.0 and y > Real.0 and p > Real.1 and q > Real.1 and Real.1 / p + Real.1 / q = Real.1 {
        log_some_of_pos_exists(x)
        let lx: Real satisfy {
            x.log = Option.some(lx)
        }
        x.log.get_or_else(Real.0) = lx
        log_some_of_pos_exists(y)
        let ly: Real satisfy {
            y.log = Option.some(ly)
        }
        y.log.get_or_else(Real.0) = ly
        if x.log.get_or_else(Real.0) <= y.log.get_or_else(Real.0) {
            if x.log.get_or_else(Real.0) < y.log.get_or_else(Real.0) {
                exp_convex_on(x.log.get_or_else(Real.0), y.log.get_or_else(Real.0))
                convex_on(Real.exp, x.log.get_or_else(Real.0), y.log.get_or_else(Real.0))
                recip_pos_of_gt_one(q)
                Real.0 < Real.1 / q
                recip_lt_one_of_gt_one(q)
                Real.1 / q < Real.1
                lt_imp_lte(Real.0, Real.1 / q)
                Real.0 <= Real.1 / q
                lt_imp_lte(Real.1 / q, Real.1)
                Real.1 / q <= Real.1
                lte_refl(x.log.get_or_else(Real.0))
                x.log.get_or_else(Real.0) <= x.log.get_or_else(Real.0)
                lte_refl(y.log.get_or_else(Real.0))
                y.log.get_or_else(Real.0) <= y.log.get_or_else(Real.0)
                lt_imp_lte(x.log.get_or_else(Real.0), y.log.get_or_else(Real.0))
                x.log.get_or_else(Real.0) <= y.log.get_or_else(Real.0)
                convex_on_apply(Real.exp, x.log.get_or_else(Real.0), y.log.get_or_else(Real.0), x.log.get_or_else(Real.0), y.log.get_or_else(Real.0), Real.1 / q)
                ((Real.1 - Real.1 / q) * x.log.get_or_else(Real.0) + (Real.1 / q) * y.log.get_or_else(Real.0)).exp <= (Real.1 - Real.1 / q) * (x.log.get_or_else(Real.0)).exp + (Real.1 / q) * (y.log.get_or_else(Real.0)).exp
                Real.1 - Real.1 / q = Real.1 / p
                ((Real.1 / p) * x.log.get_or_else(Real.0) + (Real.1 / q) * y.log.get_or_else(Real.0)).exp <= (Real.1 / p) * (x.log.get_or_else(Real.0)).exp + (Real.1 / q) * (y.log.get_or_else(Real.0)).exp
                (Real.1 / p) * x.log.get_or_else(Real.0) = x.log.get_or_else(Real.0) / p
                (Real.1 / q) * y.log.get_or_else(Real.0) = y.log.get_or_else(Real.0) / q
                (Real.1 / p) * x.log.get_or_else(Real.0) + (Real.1 / q) * y.log.get_or_else(Real.0) = x.log.get_or_else(Real.0) / p + y.log.get_or_else(Real.0) / q
                ((Real.1 / p) * x.log.get_or_else(Real.0) + (Real.1 / q) * y.log.get_or_else(Real.0)).exp = (x.log.get_or_else(Real.0) / p + y.log.get_or_else(Real.0) / q).exp
                exp_log_or_zero(x, lx)
                lx.exp = x
                (x.log.get_or_else(Real.0)).exp = x
                exp_log_or_zero(y, ly)
                ly.exp = y
                (y.log.get_or_else(Real.0)).exp = y
                (Real.1 / p) * (x.log.get_or_else(Real.0)).exp + (Real.1 / q) * (y.log.get_or_else(Real.0)).exp = x / p + y / q
                (x.log.get_or_else(Real.0) / p + y.log.get_or_else(Real.0) / q).exp <= x / p + y / q
            } else {
                not x.log.get_or_else(Real.0) < y.log.get_or_else(Real.0)
                x.log.get_or_else(Real.0) <= y.log.get_or_else(Real.0)
                lte_antisymm(x.log.get_or_else(Real.0), y.log.get_or_else(Real.0))
                x.log.get_or_else(Real.0) = y.log.get_or_else(Real.0)
                x.log.get_or_else(Real.0) / p + y.log.get_or_else(Real.0) / q = x.log.get_or_else(Real.0) / p + x.log.get_or_else(Real.0) / q
                x.log.get_or_else(Real.0) / p + x.log.get_or_else(Real.0) / q = x.log.get_or_else(Real.0) * (Real.1 / p + Real.1 / q)
                Real.1 / p + Real.1 / q = Real.1
                x.log.get_or_else(Real.0) * Real.1 = x.log.get_or_else(Real.0)
                x.log.get_or_else(Real.0) / p + y.log.get_or_else(Real.0) / q = x.log.get_or_else(Real.0)
                (x.log.get_or_else(Real.0) / p + y.log.get_or_else(Real.0) / q).exp = (x.log.get_or_else(Real.0)).exp
                exp_log_or_zero(x, lx)
                lx.exp = x
                (x.log.get_or_else(Real.0)).exp = x
                exp_log_or_zero(y, ly)
                ly.exp = y
                (y.log.get_or_else(Real.0)).exp = y
                (x.log.get_or_else(Real.0)).exp = (y.log.get_or_else(Real.0)).exp
                x = y
                (x.log.get_or_else(Real.0) / p + y.log.get_or_else(Real.0) / q).exp = x
                x / p + y / q = x / p + x / q
                x / p + x / q = x * (Real.1 / p + Real.1 / q)
                x * (Real.1 / p + Real.1 / q) = x
                x / p + y / q = x
                lte_refl(x)
                x <= x
                (x.log.get_or_else(Real.0) / p + y.log.get_or_else(Real.0) / q).exp <= x / p + y / q
            }
        } else {
            not_lte_imp_gt(x.log.get_or_else(Real.0), y.log.get_or_else(Real.0))
            x.log.get_or_else(Real.0) > y.log.get_or_else(Real.0)
            y.log.get_or_else(Real.0) < x.log.get_or_else(Real.0)
            exp_convex_on(y.log.get_or_else(Real.0), x.log.get_or_else(Real.0))
            convex_on(Real.exp, y.log.get_or_else(Real.0), x.log.get_or_else(Real.0))
            recip_pos_of_gt_one(p)
            Real.0 < Real.1 / p
            recip_lt_one_of_gt_one(p)
            Real.1 / p < Real.1
            lt_imp_lte(Real.0, Real.1 / p)
            Real.0 <= Real.1 / p
            lt_imp_lte(Real.1 / p, Real.1)
            Real.1 / p <= Real.1
            lte_refl(y.log.get_or_else(Real.0))
            y.log.get_or_else(Real.0) <= y.log.get_or_else(Real.0)
            lte_refl(x.log.get_or_else(Real.0))
            x.log.get_or_else(Real.0) <= x.log.get_or_else(Real.0)
            lt_imp_lte(y.log.get_or_else(Real.0), x.log.get_or_else(Real.0))
            y.log.get_or_else(Real.0) <= x.log.get_or_else(Real.0)
            convex_on_apply(Real.exp, y.log.get_or_else(Real.0), x.log.get_or_else(Real.0), y.log.get_or_else(Real.0), x.log.get_or_else(Real.0), Real.1 / p)
            ((Real.1 - Real.1 / p) * y.log.get_or_else(Real.0) + (Real.1 / p) * x.log.get_or_else(Real.0)).exp <= (Real.1 - Real.1 / p) * (y.log.get_or_else(Real.0)).exp + (Real.1 / p) * (x.log.get_or_else(Real.0)).exp
            Real.1 - Real.1 / p = Real.1 / q
            ((Real.1 / q) * y.log.get_or_else(Real.0) + (Real.1 / p) * x.log.get_or_else(Real.0)).exp <= (Real.1 / q) * (y.log.get_or_else(Real.0)).exp + (Real.1 / p) * (x.log.get_or_else(Real.0)).exp
            (Real.1 / p) * x.log.get_or_else(Real.0) = x.log.get_or_else(Real.0) / p
            (Real.1 / q) * y.log.get_or_else(Real.0) = y.log.get_or_else(Real.0) / q
            (Real.1 / q) * y.log.get_or_else(Real.0) + (Real.1 / p) * x.log.get_or_else(Real.0) = x.log.get_or_else(Real.0) / p + y.log.get_or_else(Real.0) / q
            ((Real.1 / q) * y.log.get_or_else(Real.0) + (Real.1 / p) * x.log.get_or_else(Real.0)).exp = (x.log.get_or_else(Real.0) / p + y.log.get_or_else(Real.0) / q).exp
            exp_log_or_zero(x, lx)
            lx.exp = x
            (x.log.get_or_else(Real.0)).exp = x
            exp_log_or_zero(y, ly)
            ly.exp = y
            (y.log.get_or_else(Real.0)).exp = y
            (Real.1 / q) * (y.log.get_or_else(Real.0)).exp + (Real.1 / p) * (x.log.get_or_else(Real.0)).exp = x / p + y / q
            (x.log.get_or_else(Real.0) / p + y.log.get_or_else(Real.0) / q).exp <= x / p + y / q
        }
    }
}

/// Young's inequality for nonnegative reals with conjugate exponents p, q > 1:
/// x^(1/p) y^(1/q) <= x/p + y/q.
theorem young_nonneg(x: Real, y: Real, p: Real, q: Real) {
    x >= Real.0 and y >= Real.0 and p > Real.1 and q > Real.1 and
        Real.1 / p + Real.1 / q = Real.1
    implies exists(uu: Real, vv: Real) {
        x.rpow(Real.1 / p) = Option.some(uu)
        and y.rpow(Real.1 / q) = Option.some(vv)
        and uu * vv <= x / p + y / q
    }
} by {
    if x >= Real.0 and y >= Real.0 and p > Real.1 and q > Real.1 and
        Real.1 / p + Real.1 / q = Real.1 {
        if x > Real.0 {
            if y > Real.0 {
                exp_convex_combine(x, y, p, q)
                (x.log.get_or_else(Real.0) / p + y.log.get_or_else(Real.0) / q).exp <= x / p + y / q
                x.rpow(Real.1 / p) = Option.some(((Real.1 / p) * x.log.get_or_else(Real.0)).exp)
                (Real.1 / p) * x.log.get_or_else(Real.0) = x.log.get_or_else(Real.0) / p
                x.rpow(Real.1 / p) = Option.some((x.log.get_or_else(Real.0) / p).exp)
                y.rpow(Real.1 / q) = Option.some(((Real.1 / q) * y.log.get_or_else(Real.0)).exp)
                (Real.1 / q) * y.log.get_or_else(Real.0) = y.log.get_or_else(Real.0) / q
                y.rpow(Real.1 / q) = Option.some((y.log.get_or_else(Real.0) / q).exp)
                exp_add(x.log.get_or_else(Real.0) / p, y.log.get_or_else(Real.0) / q)
                (x.log.get_or_else(Real.0) / p + y.log.get_or_else(Real.0) / q).exp = (x.log.get_or_else(Real.0) / p).exp * (y.log.get_or_else(Real.0) / q).exp
                exists(uu: Real, vv: Real) {
                    x.rpow(Real.1 / p) = Option.some(uu)
                    and y.rpow(Real.1 / q) = Option.some(vv)
                    and uu * vv <= x / p + y / q
                }
            } else {
                not y > Real.0
                not_gt_imp_lte(y, Real.0)
                y <= Real.0
                y >= Real.0
                lte_antisymm(y, Real.0)
                y = Real.0
                recip_pos_of_gt_one(q)
                Real.1 / q > Real.0
                rpow_nonneg_base_value(x, Real.1 / p)
                let (u0: Real) satisfy {
                    x.rpow(Real.1 / p) = Option.some(u0) and u0 >= Real.0
                }
                rpow_zero_base_pos(Real.1 / q)
                (Real.0).rpow(Real.1 / q) = Option.some(Real.0)
                y.rpow(Real.1 / q) = Option.some(Real.0)
                u0 * Real.0 = Real.0
                Real.0 < Real.1
                lt_trans(Real.0, Real.1, p)
                Real.0 < p
                p > Real.0
                div_nonneg_of_nonneg_pos(x, p)
                x / p >= Real.0
                y / q = Real.0
                x / p + y / q = x / p
                Real.0 <= x / p + y / q
                u0 * Real.0 <= x / p + y / q
                exists(uu: Real, vv: Real) {
                    x.rpow(Real.1 / p) = Option.some(uu)
                    and y.rpow(Real.1 / q) = Option.some(vv)
                    and uu * vv <= x / p + y / q
                }
            }
        } else {
            not x > Real.0
            not_gt_imp_lte(x, Real.0)
            x <= Real.0
            x >= Real.0
            lte_antisymm(x, Real.0)
            x = Real.0
            recip_pos_of_gt_one(p)
            Real.1 / p > Real.0
            rpow_nonneg_base_value(y, Real.1 / q)
            let (v0: Real) satisfy {
                y.rpow(Real.1 / q) = Option.some(v0) and v0 >= Real.0
            }
            rpow_zero_base_pos(Real.1 / p)
            (Real.0).rpow(Real.1 / p) = Option.some(Real.0)
            x.rpow(Real.1 / p) = Option.some(Real.0)
            Real.0 * v0 = Real.0
            Real.0 < Real.1
            lt_trans(Real.0, Real.1, q)
            Real.0 < q
            q > Real.0
            div_nonneg_of_nonneg_pos(y, q)
            y / q >= Real.0
            x / p = Real.0
            x / p + y / q = y / q
            Real.0 <= x / p + y / q
            Real.0 * v0 <= x / p + y / q
            exists(uu: Real, vv: Real) {
                x.rpow(Real.1 / p) = Option.some(uu)
                and y.rpow(Real.1 / q) = Option.some(vv)
                and uu * vv <= x / p + y / q
            }
        }
    }
}

// =====================================================================
// Hölder's inequality
// =====================================================================

/// The p-norm power sum of the absolute values is nonnegative.
theorem abs_pow_sum_nonneg(a: Nat -> Real, p: Real, n: Nat) {
    p > Real.0 implies abs_pow_sum(a, p, n) >= Real.0
} by {
    if p > Real.0 {
        forall(i: Nat) {
            abs_pow_value_nonneg(a, p, i)
            abs_pow_value(a, p, i) >= Real.0
            Real.0 <= abs_pow_value(a, p, i)
        }
        is_lower_bound(abs_pow_value(a, p), Real.0)
        partial_nonneg(abs_pow_value(a, p), n)
        abs_pow_sum(a, p, n) = partial(abs_pow_value(a, p), n)
        abs_pow_sum(a, p, n) >= Real.0
    }
}

/// The sequence of p-th powers of absolute values normalized by their sum.
define norm_abs_pow(a: Nat -> Real, p: Real, aa: Real, i: Nat) -> Real {
    abs_pow_value(a, p, i) / aa
}

/// The partial sum of a normalized sequence of p-th powers is one.
theorem norm_abs_pow_sum_eq_one(a: Nat -> Real, p: Real, n: Nat) {
    abs_pow_sum(a, p, n) > Real.0 implies
        partial(norm_abs_pow(a, p, abs_pow_sum(a, p, n)), n) = Real.1
} by {
    if abs_pow_sum(a, p, n) > Real.0 {
        forall(i: Nat) {
            if i < n {
                norm_abs_pow(a, p, abs_pow_sum(a, p, n), i) = abs_pow_value(a, p, i) / abs_pow_sum(a, p, n)
                abs_pow_value(a, p, i) / abs_pow_sum(a, p, n) = (Real.1 / abs_pow_sum(a, p, n)) * abs_pow_value(a, p, i)
                mul_fn(Real.1 / abs_pow_sum(a, p, n), abs_pow_value(a, p), i) = (Real.1 / abs_pow_sum(a, p, n)) * abs_pow_value(a, p, i)
                norm_abs_pow(a, p, abs_pow_sum(a, p, n), i) = mul_fn(Real.1 / abs_pow_sum(a, p, n), abs_pow_value(a, p), i)
            }
        }
        partial_pointwise_eq(norm_abs_pow(a, p, abs_pow_sum(a, p, n)), mul_fn(Real.1 / abs_pow_sum(a, p, n), abs_pow_value(a, p)), n)
        partial(norm_abs_pow(a, p, abs_pow_sum(a, p, n)), n) = partial(mul_fn(Real.1 / abs_pow_sum(a, p, n), abs_pow_value(a, p)), n)
        partial_scalar_mul(Real.1 / abs_pow_sum(a, p, n), abs_pow_value(a, p), n)
        (Real.1 / abs_pow_sum(a, p, n)) * partial(abs_pow_value(a, p), n) = partial(mul_fn(Real.1 / abs_pow_sum(a, p, n), abs_pow_value(a, p)), n)
        partial(norm_abs_pow(a, p, abs_pow_sum(a, p, n)), n) = (Real.1 / abs_pow_sum(a, p, n)) * partial(abs_pow_value(a, p), n)
        partial(abs_pow_value(a, p), n) = abs_pow_sum(a, p, n)
        partial(norm_abs_pow(a, p, abs_pow_sum(a, p, n)), n) = (Real.1 / abs_pow_sum(a, p, n)) * abs_pow_sum(a, p, n)
        mul_div_cancel(Real.1, abs_pow_sum(a, p, n))
        abs_pow_sum(a, p, n) * (Real.1 / abs_pow_sum(a, p, n)) = Real.1
        (Real.1 / abs_pow_sum(a, p, n)) * abs_pow_sum(a, p, n) = Real.1
        partial(norm_abs_pow(a, p, abs_pow_sum(a, p, n)), n) = Real.1
    }
}

/// The first Young summand of Hölder's inequality, pointwise.
define holder_rhs_first(a: Nat -> Real, p: Real, aa: Real, i: Nat) -> Real {
    (abs_pow_value(a, p, i) / aa) / p
}

/// The second Young summand of Hölder's inequality, pointwise.
define holder_rhs_second(b: Nat -> Real, q: Real, bb: Real, i: Nat) -> Real {
    (abs_pow_value(b, q, i) / bb) / q
}

/// The pointwise right-hand side in Hölder's inequality: the normalized
/// Young summands.
define holder_rhs(a: Nat -> Real, b: Nat -> Real, p: Real, q: Real, aa: Real, bb: Real, i: Nat) -> Real {
    holder_rhs_first(a, p, aa, i) + holder_rhs_second(b, q, bb, i)
}

/// The partial sum of the first Young summand is 1/p.
theorem holder_rhs_first_sum(a: Nat -> Real, n: Nat, p: Real) {
    p > Real.0 and abs_pow_sum(a, p, n) > Real.0 implies
        partial(holder_rhs_first(a, p, abs_pow_sum(a, p, n)), n) = Real.1 / p
} by {
    if p > Real.0 and abs_pow_sum(a, p, n) > Real.0 {
        forall(i: Nat) {
            if i < n {
                holder_rhs_first(a, p, abs_pow_sum(a, p, n), i) = (abs_pow_value(a, p, i) / abs_pow_sum(a, p, n)) / p
                (abs_pow_value(a, p, i) / abs_pow_sum(a, p, n)) / p = (Real.1 / p) * (abs_pow_value(a, p, i) / abs_pow_sum(a, p, n))
                mul_fn(Real.1 / p, norm_abs_pow(a, p, abs_pow_sum(a, p, n)), i) = (Real.1 / p) * (abs_pow_value(a, p, i) / abs_pow_sum(a, p, n))
                holder_rhs_first(a, p, abs_pow_sum(a, p, n), i) = mul_fn(Real.1 / p, norm_abs_pow(a, p, abs_pow_sum(a, p, n)), i)
            }
        }
        partial_pointwise_eq(holder_rhs_first(a, p, abs_pow_sum(a, p, n)), mul_fn(Real.1 / p, norm_abs_pow(a, p, abs_pow_sum(a, p, n))), n)
        partial(holder_rhs_first(a, p, abs_pow_sum(a, p, n)), n) = partial(mul_fn(Real.1 / p, norm_abs_pow(a, p, abs_pow_sum(a, p, n))), n)
        partial_scalar_mul(Real.1 / p, norm_abs_pow(a, p, abs_pow_sum(a, p, n)), n)
        (Real.1 / p) * partial(norm_abs_pow(a, p, abs_pow_sum(a, p, n)), n) = partial(mul_fn(Real.1 / p, norm_abs_pow(a, p, abs_pow_sum(a, p, n))), n)
        partial(holder_rhs_first(a, p, abs_pow_sum(a, p, n)), n) = (Real.1 / p) * partial(norm_abs_pow(a, p, abs_pow_sum(a, p, n)), n)
        norm_abs_pow_sum_eq_one(a, p, n)
        partial(norm_abs_pow(a, p, abs_pow_sum(a, p, n)), n) = Real.1
        (Real.1 / p) * Real.1 = Real.1 / p
        partial(holder_rhs_first(a, p, abs_pow_sum(a, p, n)), n) = Real.1 / p
    }
}

/// The partial sum of the second Young summand is 1/q.
theorem holder_rhs_second_sum(b: Nat -> Real, n: Nat, q: Real) {
    q > Real.0 and abs_pow_sum(b, q, n) > Real.0 implies
        partial(holder_rhs_second(b, q, abs_pow_sum(b, q, n)), n) = Real.1 / q
} by {
    if q > Real.0 and abs_pow_sum(b, q, n) > Real.0 {
        forall(i: Nat) {
            if i < n {
                holder_rhs_second(b, q, abs_pow_sum(b, q, n), i) = (abs_pow_value(b, q, i) / abs_pow_sum(b, q, n)) / q
                (abs_pow_value(b, q, i) / abs_pow_sum(b, q, n)) / q = (Real.1 / q) * (abs_pow_value(b, q, i) / abs_pow_sum(b, q, n))
                mul_fn(Real.1 / q, norm_abs_pow(b, q, abs_pow_sum(b, q, n)), i) = (Real.1 / q) * (abs_pow_value(b, q, i) / abs_pow_sum(b, q, n))
                holder_rhs_second(b, q, abs_pow_sum(b, q, n), i) = mul_fn(Real.1 / q, norm_abs_pow(b, q, abs_pow_sum(b, q, n)), i)
            }
        }
        partial_pointwise_eq(holder_rhs_second(b, q, abs_pow_sum(b, q, n)), mul_fn(Real.1 / q, norm_abs_pow(b, q, abs_pow_sum(b, q, n))), n)
        partial(holder_rhs_second(b, q, abs_pow_sum(b, q, n)), n) = partial(mul_fn(Real.1 / q, norm_abs_pow(b, q, abs_pow_sum(b, q, n))), n)
        partial_scalar_mul(Real.1 / q, norm_abs_pow(b, q, abs_pow_sum(b, q, n)), n)
        (Real.1 / q) * partial(norm_abs_pow(b, q, abs_pow_sum(b, q, n)), n) = partial(mul_fn(Real.1 / q, norm_abs_pow(b, q, abs_pow_sum(b, q, n))), n)
        partial(holder_rhs_second(b, q, abs_pow_sum(b, q, n)), n) = (Real.1 / q) * partial(norm_abs_pow(b, q, abs_pow_sum(b, q, n)), n)
        norm_abs_pow_sum_eq_one(b, q, n)
        partial(norm_abs_pow(b, q, abs_pow_sum(b, q, n)), n) = Real.1
        (Real.1 / q) * Real.1 = Real.1 / q
        partial(holder_rhs_second(b, q, abs_pow_sum(b, q, n)), n) = Real.1 / q
    }
}

/// The partial sum of the normalized Young summands is one.
theorem holder_rhs_sum_eq_one(a: Nat -> Real, b: Nat -> Real, n: Nat, p: Real, q: Real) {
    p > Real.0 and q > Real.0 and Real.1 / p + Real.1 / q = Real.1
    and abs_pow_sum(a, p, n) > Real.0 and abs_pow_sum(b, q, n) > Real.0
    implies partial(holder_rhs(a, b, p, q, abs_pow_sum(a, p, n), abs_pow_sum(b, q, n)), n) = Real.1
} by {
    if p > Real.0 and q > Real.0 and Real.1 / p + Real.1 / q = Real.1
        and abs_pow_sum(a, p, n) > Real.0 and abs_pow_sum(b, q, n) > Real.0 {
        forall(i: Nat) {
            if i < n {
                holder_rhs(a, b, p, q, abs_pow_sum(a, p, n), abs_pow_sum(b, q, n), i) = holder_rhs_first(a, p, abs_pow_sum(a, p, n), i) + holder_rhs_second(b, q, abs_pow_sum(b, q, n), i)
                add_fn(holder_rhs_first(a, p, abs_pow_sum(a, p, n)), holder_rhs_second(b, q, abs_pow_sum(b, q, n)), i) = holder_rhs_first(a, p, abs_pow_sum(a, p, n), i) + holder_rhs_second(b, q, abs_pow_sum(b, q, n), i)
                holder_rhs(a, b, p, q, abs_pow_sum(a, p, n), abs_pow_sum(b, q, n), i) = add_fn(holder_rhs_first(a, p, abs_pow_sum(a, p, n)), holder_rhs_second(b, q, abs_pow_sum(b, q, n)), i)
            }
        }
        partial_pointwise_eq(holder_rhs(a, b, p, q, abs_pow_sum(a, p, n), abs_pow_sum(b, q, n)), add_fn(holder_rhs_first(a, p, abs_pow_sum(a, p, n)), holder_rhs_second(b, q, abs_pow_sum(b, q, n))), n)
        partial(holder_rhs(a, b, p, q, abs_pow_sum(a, p, n), abs_pow_sum(b, q, n)), n) = partial(add_fn(holder_rhs_first(a, p, abs_pow_sum(a, p, n)), holder_rhs_second(b, q, abs_pow_sum(b, q, n))), n)
        partial_add(holder_rhs_first(a, p, abs_pow_sum(a, p, n)), holder_rhs_second(b, q, abs_pow_sum(b, q, n)), n)
        partial(holder_rhs_first(a, p, abs_pow_sum(a, p, n)), n) + partial(holder_rhs_second(b, q, abs_pow_sum(b, q, n)), n) = partial(add_fn(holder_rhs_first(a, p, abs_pow_sum(a, p, n)), holder_rhs_second(b, q, abs_pow_sum(b, q, n))), n)
        partial(holder_rhs(a, b, p, q, abs_pow_sum(a, p, n), abs_pow_sum(b, q, n)), n) = partial(holder_rhs_first(a, p, abs_pow_sum(a, p, n)), n) + partial(holder_rhs_second(b, q, abs_pow_sum(b, q, n)), n)
        holder_rhs_first_sum(a, n, p)
        partial(holder_rhs_first(a, p, abs_pow_sum(a, p, n)), n) = Real.1 / p
        holder_rhs_second_sum(b, n, q)
        partial(holder_rhs_second(b, q, abs_pow_sum(b, q, n)), n) = Real.1 / q
        partial(holder_rhs(a, b, p, q, abs_pow_sum(a, p, n), abs_pow_sum(b, q, n)), n) = Real.1 / p + Real.1 / q
        Real.1 / p + Real.1 / q = Real.1
        partial(holder_rhs(a, b, p, q, abs_pow_sum(a, p, n), abs_pow_sum(b, q, n)), n) = Real.1
    }
}

/// The pointwise left-hand side in Hölder's inequality, normalized by the
/// p-norm and q-norm values.
define holder_lhs(a: Nat -> Real, b: Nat -> Real, uu: Real, vv: Real, i: Nat) -> Real {
    (a(i).abs / uu) * (b(i).abs / vv)
}

/// The partial sum of the normalized left-hand side is the original sum
/// divided by the product of the norms.
theorem holder_lhs_scaled(a: Nat -> Real, b: Nat -> Real, n: Nat, uu: Real, vv: Real) {
    uu > Real.0 and vv > Real.0
    implies partial(holder_lhs(a, b, uu, vv), n) = (Real.1 / (uu * vv)) * partial(abs_product(a, b), n)
} by {
    if uu > Real.0 and vv > Real.0 {
        forall(i: Nat) {
            if i < n {
                holder_lhs(a, b, uu, vv, i) = (a(i).abs / uu) * (b(i).abs / vv)
                uu != Real.0
                vv != Real.0
                mul_div(a(i).abs, uu, b(i).abs, vv)
                (a(i).abs / uu) * (b(i).abs / vv) = (a(i).abs * b(i).abs) / (uu * vv)
                mul_abs(a(i), b(i))
                a(i).abs * b(i).abs = (a(i) * b(i)).abs
                (a(i).abs * b(i).abs) / (uu * vv) = (a(i) * b(i)).abs / (uu * vv)
                (a(i).abs / uu) * (b(i).abs / vv) = (a(i) * b(i)).abs / (uu * vv)
                (a(i) * b(i)).abs / (uu * vv) = (Real.1 / (uu * vv)) * (a(i) * b(i)).abs
                mul_fn(Real.1 / (uu * vv), abs_product(a, b), i) = (Real.1 / (uu * vv)) * (a(i) * b(i)).abs
                holder_lhs(a, b, uu, vv, i) = mul_fn(Real.1 / (uu * vv), abs_product(a, b), i)
            }
        }
        partial_pointwise_eq(holder_lhs(a, b, uu, vv), mul_fn(Real.1 / (uu * vv), abs_product(a, b)), n)
        partial(holder_lhs(a, b, uu, vv), n) = partial(mul_fn(Real.1 / (uu * vv), abs_product(a, b)), n)
        partial_scalar_mul(Real.1 / (uu * vv), abs_product(a, b), n)
        (Real.1 / (uu * vv)) * partial(abs_product(a, b), n) = partial(mul_fn(Real.1 / (uu * vv), abs_product(a, b)), n)
        partial(holder_lhs(a, b, uu, vv), n) = (Real.1 / (uu * vv)) * partial(abs_product(a, b), n)
    }
}

/// If the sum of the p-th powers of the absolute values of `a` vanishes,
/// then the sum of |a_i b_i| vanishes.
theorem abs_product_sum_zero_of_abs_pow_sum_zero(a: Nat -> Real, b: Nat -> Real, n: Nat, p: Real) {
    p > Real.0 and abs_pow_sum(a, p, n) = Real.0 implies partial(abs_product(a, b), n) = Real.0
} by {
    if p > Real.0 and abs_pow_sum(a, p, n) = Real.0 {
        forall(i: Nat) {
            abs_pow_value_nonneg(a, p, i)
            abs_pow_value(a, p, i) >= Real.0
            Real.0 <= abs_pow_value(a, p, i)
        }
        is_lower_bound(abs_pow_value(a, p), Real.0)
        define pred(k: Nat) -> Bool {
            abs_pow_sum(a, p, k) = Real.0 implies partial(abs_product(a, b), k) = Real.0
        }
        partial_zero(abs_product(a, b))
        partial(abs_product(a, b), Nat.0) = Real.0
        pred(Nat.0)
        forall(k: Nat) {
            if pred(k) {
                if abs_pow_sum(a, p, k.suc) = Real.0 {
                    partial(abs_pow_value(a, p), k.suc) = abs_pow_sum(a, p, k.suc)
                    partial(abs_pow_value(a, p), k.suc) = Real.0
                    partial_split_last(abs_pow_value(a, p), k)
                    partial(abs_pow_value(a, p), k.suc) = partial(abs_pow_value(a, p), k) + abs_pow_value(a, p, k)
                    partial(abs_pow_value(a, p), k) + abs_pow_value(a, p, k) = Real.0
                    partial_nonneg(abs_pow_value(a, p), k)
                    partial(abs_pow_value(a, p), k) >= Real.0
                    abs_pow_value_nonneg(a, p, k)
                    abs_pow_value(a, p, k) >= Real.0
                    add_nonneg_eq_zero_left(partial(abs_pow_value(a, p), k), abs_pow_value(a, p, k))
                    partial(abs_pow_value(a, p), k) = Real.0
                    add_nonneg_eq_zero_right(partial(abs_pow_value(a, p), k), abs_pow_value(a, p, k))
                    abs_pow_value(a, p, k) = Real.0
                    abs_pow_value_zero_imp_abs_zero(a, p, k)
                    a(k).abs = Real.0
                    only_abs_zero_eq_zero(a(k))
                    a(k) = Real.0
                    a(k) * b(k) = Real.0
                    (a(k) * b(k)).abs = Real.0
                    abs_product(a, b, k) = Real.0
                    pred(k) = (abs_pow_sum(a, p, k) = Real.0 implies partial(abs_product(a, b), k) = Real.0)
                    abs_pow_sum(a, p, k) = partial(abs_pow_value(a, p), k)
                    abs_pow_sum(a, p, k) = Real.0
                    partial(abs_product(a, b), k) = Real.0
                    partial_split_last(abs_product(a, b), k)
                    partial(abs_product(a, b), k.suc) = partial(abs_product(a, b), k) + abs_product(a, b, k)
                    partial(abs_product(a, b), k) + abs_product(a, b, k) = Real.0
                    partial(abs_product(a, b), k.suc) = Real.0
                }
                pred(k.suc)
            }
        }
        pred(Nat.0) and forall(k: Nat) { pred(k) implies pred(k.suc) }
        alt_induction(pred)
        forall(k: Nat) { pred(k) }
        pred(n)
        pred(n) = (abs_pow_sum(a, p, n) = Real.0 implies partial(abs_product(a, b), n) = Real.0)
        partial(abs_product(a, b), n) = Real.0
    }
}

/// The p-norm root is zero when the norm sum is zero.
theorem rpow_root_zero_of_sum_zero(a: Nat -> Real, n: Nat, p: Real, uu: Real) {
    p > Real.0 and abs_pow_sum(a, p, n) = Real.0 and (abs_pow_sum(a, p, n)).rpow(Real.1 / p) = Option.some(uu)
    implies uu = Real.0
} by {
    if p > Real.0 and abs_pow_sum(a, p, n) = Real.0 and (abs_pow_sum(a, p, n)).rpow(Real.1 / p) = Option.some(uu) {
        Real.1 / p > Real.0
        rpow_zero_base_pos(Real.1 / p)
        (Real.0).rpow(Real.1 / p) = Option.some(Real.0)
        (abs_pow_sum(a, p, n)).rpow(Real.1 / p) = Option.some(Real.0)
        some_injective[Real](uu, Real.0)
        uu = Real.0
    }
}

/// The sum of |a_i b_i| is symmetric in the two sequences.
theorem abs_product_symm(a: Nat -> Real, b: Nat -> Real, n: Nat) {
    partial(abs_product(a, b), n) = partial(abs_product(b, a), n)
} by {
    forall(i: Nat) {
        if i < n {
            abs_product(a, b, i) = (a(i) * b(i)).abs
            abs_product(b, a, i) = (b(i) * a(i)).abs
            a(i) * b(i) = b(i) * a(i)
            (a(i) * b(i)).abs = (b(i) * a(i)).abs
            abs_product(a, b, i) = abs_product(b, a, i)
        }
    }
    partial_pointwise_eq(abs_product(a, b), abs_product(b, a), n)
    partial(abs_product(a, b), n) = partial(abs_product(b, a), n)
}

/// Hölder's inequality for finite real sequences, value form: for conjugate
/// exponents p, q > 1 with uu the p-norm of `a` and vv the q-norm of `b`,
/// the sum of |a_i b_i| is at most uu * vv.
theorem finite_holder(a: Nat -> Real, b: Nat -> Real, n: Nat, p: Real, q: Real, uu: Real, vv: Real) {
    p > Real.1 and q > Real.1 and Real.1 / p + Real.1 / q = Real.1
    and (abs_pow_sum(a, p, n)).rpow(Real.1 / p) = Option.some(uu)
    and (abs_pow_sum(b, q, n)).rpow(Real.1 / q) = Option.some(vv)
    implies partial(abs_product(a, b), n) <= uu * vv
} by {
    if p > Real.1 and q > Real.1 and Real.1 / p + Real.1 / q = Real.1
        and (abs_pow_sum(a, p, n)).rpow(Real.1 / p) = Option.some(uu)
        and (abs_pow_sum(b, q, n)).rpow(Real.1 / q) = Option.some(vv) {
        gt_one_imp_pos(p)
        p > Real.0
        abs_pow_sum_nonneg(a, p, n)
        abs_pow_sum(a, p, n) >= Real.0
        gt_one_imp_pos(q)
        q > Real.0
        abs_pow_sum_nonneg(b, q, n)
        abs_pow_sum(b, q, n) >= Real.0
        if abs_pow_sum(a, p, n) > Real.0 {
            if abs_pow_sum(b, q, n) > Real.0 {
                (abs_pow_sum(a, p, n)).rpow(Real.1 / p) = Option.some(((Real.1 / p) * (abs_pow_sum(a, p, n)).log.get_or_else(Real.0)).exp)
                Option.some(uu) = Option.some(((Real.1 / p) * (abs_pow_sum(a, p, n)).log.get_or_else(Real.0)).exp)
                some_injective[Real](uu, ((Real.1 / p) * (abs_pow_sum(a, p, n)).log.get_or_else(Real.0)).exp)
                uu = ((Real.1 / p) * (abs_pow_sum(a, p, n)).log.get_or_else(Real.0)).exp
                exp_pos((Real.1 / p) * (abs_pow_sum(a, p, n)).log.get_or_else(Real.0))
                ((Real.1 / p) * (abs_pow_sum(a, p, n)).log.get_or_else(Real.0)).exp > Real.0
                uu > Real.0
                (abs_pow_sum(b, q, n)).rpow(Real.1 / q) = Option.some(((Real.1 / q) * (abs_pow_sum(b, q, n)).log.get_or_else(Real.0)).exp)
                Option.some(vv) = Option.some(((Real.1 / q) * (abs_pow_sum(b, q, n)).log.get_or_else(Real.0)).exp)
                some_injective[Real](vv, ((Real.1 / q) * (abs_pow_sum(b, q, n)).log.get_or_else(Real.0)).exp)
                vv = ((Real.1 / q) * (abs_pow_sum(b, q, n)).log.get_or_else(Real.0)).exp
                exp_pos((Real.1 / q) * (abs_pow_sum(b, q, n)).log.get_or_else(Real.0))
                ((Real.1 / q) * (abs_pow_sum(b, q, n)).log.get_or_else(Real.0)).exp > Real.0
                vv > Real.0
                forall(i: Nat) {
                    if i < n {
                        p > Real.0
                        abs_pow_value_nonneg(a, p, i)
                        abs_pow_value(a, p, i) >= Real.0
                        q > Real.0
                        abs_pow_value_nonneg(b, q, i)
                        abs_pow_value(b, q, i) >= Real.0
                        div_nonneg_of_nonneg_pos(abs_pow_value(a, p, i), abs_pow_sum(a, p, n))
                        abs_pow_value(a, p, i) / abs_pow_sum(a, p, n) >= Real.0
                        div_nonneg_of_nonneg_pos(abs_pow_value(b, q, i), abs_pow_sum(b, q, n))
                        abs_pow_value(b, q, i) / abs_pow_sum(b, q, n) >= Real.0
                        young_nonneg(abs_pow_value(a, p, i) / abs_pow_sum(a, p, n), abs_pow_value(b, q, i) / abs_pow_sum(b, q, n), p, q)
                        let (u1: Real, v1: Real) satisfy {
                            (abs_pow_value(a, p, i) / abs_pow_sum(a, p, n)).rpow(Real.1 / p) = Option.some(u1)
                            and (abs_pow_value(b, q, i) / abs_pow_sum(b, q, n)).rpow(Real.1 / q) = Option.some(v1)
                            and u1 * v1 <= (abs_pow_value(a, p, i) / abs_pow_sum(a, p, n)) / p + (abs_pow_value(b, q, i) / abs_pow_sum(b, q, n)) / q
                        }
                        normalized_abs_pow_root_eq(a, p, abs_pow_sum(a, p, n), uu, i)
                        (abs_pow_value(a, p, i) / abs_pow_sum(a, p, n)).rpow(Real.1 / p) = Option.some(a(i).abs / uu)
                        some_injective[Real](u1, a(i).abs / uu)
                        u1 = a(i).abs / uu
                        normalized_abs_pow_root_eq(b, q, abs_pow_sum(b, q, n), vv, i)
                        (abs_pow_value(b, q, i) / abs_pow_sum(b, q, n)).rpow(Real.1 / q) = Option.some(b(i).abs / vv)
                        some_injective[Real](v1, b(i).abs / vv)
                        v1 = b(i).abs / vv
                        u1 * v1 = (a(i).abs / uu) * (b(i).abs / vv)
                        (a(i).abs / uu) * (b(i).abs / vv) <= (abs_pow_value(a, p, i) / abs_pow_sum(a, p, n)) / p + (abs_pow_value(b, q, i) / abs_pow_sum(b, q, n)) / q
                        holder_lhs(a, b, uu, vv, i) = (a(i).abs / uu) * (b(i).abs / vv)
                        holder_rhs(a, b, p, q, abs_pow_sum(a, p, n), abs_pow_sum(b, q, n), i) = holder_rhs_first(a, p, abs_pow_sum(a, p, n), i) + holder_rhs_second(b, q, abs_pow_sum(b, q, n), i)
                        holder_rhs_first(a, p, abs_pow_sum(a, p, n), i) = (abs_pow_value(a, p, i) / abs_pow_sum(a, p, n)) / p
                        holder_rhs_second(b, q, abs_pow_sum(b, q, n), i) = (abs_pow_value(b, q, i) / abs_pow_sum(b, q, n)) / q
                        holder_rhs(a, b, p, q, abs_pow_sum(a, p, n), abs_pow_sum(b, q, n), i) = (abs_pow_value(a, p, i) / abs_pow_sum(a, p, n)) / p + (abs_pow_value(b, q, i) / abs_pow_sum(b, q, n)) / q
                        holder_lhs(a, b, uu, vv, i) <= holder_rhs(a, b, p, q, abs_pow_sum(a, p, n), abs_pow_sum(b, q, n), i)
                    }
                }
                partial_le_range(holder_lhs(a, b, uu, vv), holder_rhs(a, b, p, q, abs_pow_sum(a, p, n), abs_pow_sum(b, q, n)), n)
                partial(holder_lhs(a, b, uu, vv), n) <= partial(holder_rhs(a, b, p, q, abs_pow_sum(a, p, n), abs_pow_sum(b, q, n)), n)
                p > Real.0
                q > Real.0
                holder_rhs_sum_eq_one(a, b, n, p, q)
                partial(holder_rhs(a, b, p, q, abs_pow_sum(a, p, n), abs_pow_sum(b, q, n)), n) = Real.1
                partial(holder_lhs(a, b, uu, vv), n) <= Real.1
                holder_lhs_scaled(a, b, n, uu, vv)
                partial(holder_lhs(a, b, uu, vv), n) = (Real.1 / (uu * vv)) * partial(abs_product(a, b), n)
                (Real.1 / (uu * vv)) * partial(abs_product(a, b), n) <= Real.1
                gt_zero_imp_pos(uu)
                uu.is_positive
                gt_zero_imp_pos(vv)
                vv.is_positive
                mul_pos_pos(uu, vv)
                (uu * vv).is_positive
                pos_gt_zero(uu * vv)
                uu * vv > Real.0
                mul_le_mul_pos_right((Real.1 / (uu * vv)) * partial(abs_product(a, b), n), Real.1, uu * vv)
                ((Real.1 / (uu * vv)) * partial(abs_product(a, b), n)) * (uu * vv) <= Real.1 * (uu * vv)
                ((Real.1 / (uu * vv)) * partial(abs_product(a, b), n)) * (uu * vv) = (uu * vv) * ((Real.1 / (uu * vv)) * partial(abs_product(a, b), n))
                (uu * vv) * ((Real.1 / (uu * vv)) * partial(abs_product(a, b), n)) = ((uu * vv) * (Real.1 / (uu * vv))) * partial(abs_product(a, b), n)
                mul_div_cancel(Real.1, uu * vv)
                (uu * vv) * (Real.1 / (uu * vv)) = Real.1
                ((uu * vv) * (Real.1 / (uu * vv))) * partial(abs_product(a, b), n) = partial(abs_product(a, b), n)
                ((Real.1 / (uu * vv)) * partial(abs_product(a, b), n)) * (uu * vv) = partial(abs_product(a, b), n)
                Real.1 * (uu * vv) = uu * vv
                partial(abs_product(a, b), n) <= uu * vv
            } else {
                not abs_pow_sum(b, q, n) > Real.0
                not_gt_imp_lte(abs_pow_sum(b, q, n), Real.0)
                abs_pow_sum(b, q, n) <= Real.0
                abs_pow_sum(b, q, n) >= Real.0
                lte_antisymm(abs_pow_sum(b, q, n), Real.0)
                abs_pow_sum(b, q, n) = Real.0
                Real.0 < Real.1
                lt_trans(Real.0, Real.1, q)
                Real.0 < q
                q > Real.0
                abs_product_sum_zero_of_abs_pow_sum_zero(b, a, n, q)
                partial(abs_product(b, a), n) = Real.0
                abs_product_symm(a, b, n)
                partial(abs_product(a, b), n) = partial(abs_product(b, a), n)
                partial(abs_product(a, b), n) = Real.0
                rpow_root_zero_of_sum_zero(b, n, q, vv)
                vv = Real.0
                recip_pos_of_gt_one(p)
                Real.1 / p > Real.0
                rpow_nonneg_base_value(abs_pow_sum(a, p, n), Real.1 / p)
                let (u0: Real) satisfy {
                    (abs_pow_sum(a, p, n)).rpow(Real.1 / p) = Option.some(u0) and u0 >= Real.0
                }
                (abs_pow_sum(a, p, n)).rpow(Real.1 / p) = Option.some(uu)
                some_injective[Real](uu, u0)
                uu = u0
                uu >= Real.0
                uu * vv = uu * Real.0
                uu * Real.0 = Real.0
                uu * vv = Real.0
                partial(abs_product(a, b), n) = Real.0
                Real.0 <= Real.0
                partial(abs_product(a, b), n) <= uu * vv
            }
        } else {
            not abs_pow_sum(a, p, n) > Real.0
            not_gt_imp_lte(abs_pow_sum(a, p, n), Real.0)
            abs_pow_sum(a, p, n) <= Real.0
            abs_pow_sum(a, p, n) >= Real.0
            lte_antisymm(abs_pow_sum(a, p, n), Real.0)
            abs_pow_sum(a, p, n) = Real.0
            Real.0 < Real.1
            lt_trans(Real.0, Real.1, p)
            Real.0 < p
            p > Real.0
            abs_product_sum_zero_of_abs_pow_sum_zero(a, b, n, p)
            partial(abs_product(a, b), n) = Real.0
            rpow_root_zero_of_sum_zero(a, n, p, uu)
            uu = Real.0
            recip_pos_of_gt_one(q)
            Real.1 / q > Real.0
            rpow_nonneg_base_value(abs_pow_sum(b, q, n), Real.1 / q)
            let (v0: Real) satisfy {
                (abs_pow_sum(b, q, n)).rpow(Real.1 / q) = Option.some(v0) and v0 >= Real.0
            }
            (abs_pow_sum(b, q, n)).rpow(Real.1 / q) = Option.some(vv)
            some_injective[Real](vv, v0)
            vv = v0
            vv >= Real.0
            uu * vv = Real.0 * vv
            Real.0 * vv = Real.0
            uu * vv = Real.0
            partial(abs_product(a, b), n) = Real.0
            Real.0 <= Real.0
            partial(abs_product(a, b), n) <= uu * vv
        }
    }
}

/// Hölder's inequality for finite real sequences: for conjugate exponents
/// p, q > 1, the sum of |a_i b_i| is at most the product of the p-norm of `a`
/// and the q-norm of `b`.
theorem finite_holder_exists(a: Nat -> Real, b: Nat -> Real, n: Nat, p: Real, q: Real) {
    p > Real.1 and q > Real.1 and Real.1 / p + Real.1 / q = Real.1
    implies exists(uu: Real, vv: Real) {
        (abs_pow_sum(a, p, n)).rpow(Real.1 / p) = Option.some(uu)
        and (abs_pow_sum(b, q, n)).rpow(Real.1 / q) = Option.some(vv)
        and partial(abs_product(a, b), n) <= uu * vv
    }
} by {
    if p > Real.1 and q > Real.1 and Real.1 / p + Real.1 / q = Real.1 {
        gt_one_imp_pos(p)
        p > Real.0
        abs_pow_sum_nonneg(a, p, n)
        abs_pow_sum(a, p, n) >= Real.0
        recip_pos_of_gt_one(p)
        Real.1 / p > Real.0
        rpow_nonneg_base_value(abs_pow_sum(a, p, n), Real.1 / p)
        let (u0: Real) satisfy {
            (abs_pow_sum(a, p, n)).rpow(Real.1 / p) = Option.some(u0) and u0 >= Real.0
        }
        gt_one_imp_pos(q)
        q > Real.0
        abs_pow_sum_nonneg(b, q, n)
        abs_pow_sum(b, q, n) >= Real.0
        recip_pos_of_gt_one(q)
        Real.1 / q > Real.0
        rpow_nonneg_base_value(abs_pow_sum(b, q, n), Real.1 / q)
        let (v0: Real) satisfy {
            (abs_pow_sum(b, q, n)).rpow(Real.1 / q) = Option.some(v0) and v0 >= Real.0
        }
        finite_holder(a, b, n, p, q, u0, v0)
        partial(abs_product(a, b), n) <= u0 * v0
        exists(uu: Real, vv: Real) {
            (abs_pow_sum(a, p, n)).rpow(Real.1 / p) = Option.some(uu)
            and (abs_pow_sum(b, q, n)).rpow(Real.1 / q) = Option.some(vv)
            and partial(abs_product(a, b), n) <= uu * vv
        }
    }
}

// =====================================================================
// Minkowski's inequality
// =====================================================================

/// The conjugate exponent of p: the unique q > 1 with 1/p + 1/q = 1.
define conjugate_exponent(p: Real) -> Real {
    p / (p - Real.1)
}

/// A real above one stays above one after subtracting one.
theorem sub_one_pos(x: Real) {
    x > Real.1 implies x - Real.1 > Real.0
} by {
    if x > Real.1 {
        lt_add_right(Real.1, x, -Real.1)
        Real.1 + -Real.1 < x + -Real.1
        add_neg_eq_zero(Real.1)
        Real.1 + -Real.1 = Real.0
        Real.0 < x + -Real.1
        x - Real.1 = x + -Real.1
        Real.0 < x - Real.1
        x - Real.1 > Real.0
    }
}


/// Adding two quotients with a common denominator.
theorem div_add_same_denom(a: Real, b: Real, p: Real) {
    p != Real.0 implies a / p + b / p = (a + b) / p
} by {
    if p != Real.0 {
        a / p + b / p = (a + b) / p
    }
}

/// The reciprocal of the conjugate exponent is (p - 1)/p.
theorem conjugate_exponent_recip_value(p: Real) {
    p > Real.1 implies Real.1 / conjugate_exponent(p) = (p - Real.1) / p
} by {
    if p > Real.1 {
        p != Real.0
        p - Real.1 != Real.0
        conjugate_exponent(p) = p / (p - Real.1)
        Real.1 / conjugate_exponent(p) = Real.1 / (p / (p - Real.1))
        div_div(Real.1, Real.1, p, p - Real.1)
        (Real.1 / Real.1) / (p / (p - Real.1)) = (Real.1 * (p - Real.1)) / (Real.1 * p)
        Real.1 / Real.1 = Real.1
        (Real.1 / Real.1) / (p / (p - Real.1)) = Real.1 / (p / (p - Real.1))
        Real.1 * (p - Real.1) = p - Real.1
        Real.1 * p = p
        (Real.1 * (p - Real.1)) / (Real.1 * p) = (p - Real.1) / p
        Real.1 / (p / (p - Real.1)) = (p - Real.1) / p
        Real.1 / conjugate_exponent(p) = (p - Real.1) / p
    }
}

/// Conjugate exponents satisfy 1/p + 1/q = 1.
theorem conjugate_exponent_recip(p: Real) {
    p > Real.1 implies Real.1 / p + Real.1 / conjugate_exponent(p) = Real.1
} by {
    if p > Real.1 {
        conjugate_exponent_recip_value(p)
        Real.1 / conjugate_exponent(p) = (p - Real.1) / p
        Real.1 / p + Real.1 / conjugate_exponent(p) = Real.1 / p + (p - Real.1) / p
        p != Real.0
        div_add_same_denom(Real.1, p - Real.1, p)
        Real.1 / p + (p - Real.1) / p = (Real.1 + (p - Real.1)) / p
        p - Real.1 = p + -Real.1
        (p - Real.1) + Real.1 = (p + -Real.1) + Real.1
        (p + -Real.1) + Real.1 = p + (-Real.1 + Real.1)
        add_neg_eq_zero(Real.1)
        Real.1 + -Real.1 = Real.0
        -Real.1 + Real.1 = Real.0
        p + (-Real.1 + Real.1) = p + Real.0
        add_zero_right(p)
        p + Real.0 = p
        (p - Real.1) + Real.1 = p
        Real.1 + (p - Real.1) = p
        (Real.1 + (p - Real.1)) / p = p / p
        mul_div_cancel(Real.1, p)
        p * (Real.1 / p) = Real.1
        p / p = Real.1
        (Real.1 + (p - Real.1)) / p = Real.1
        Real.1 / p + Real.1 / conjugate_exponent(p) = Real.1
    }
}

/// The conjugate exponent of p is above one.
theorem conjugate_exponent_gt_one(p: Real) {
    p > Real.1 implies conjugate_exponent(p) > Real.1
} by {
    if p > Real.1 {
        conjugate_exponent_recip_value(p)
        Real.1 / conjugate_exponent(p) = (p - Real.1) / p
        sub_one_pos(p)
        p - Real.1 > Real.0
        Real.0 < Real.1
        lt_trans(Real.0, Real.1, p)
        Real.0 < p
        p > Real.0
        Real.0 < Real.1
        neg_lt_swap_neg(Real.0, Real.1)
        -Real.1 < -Real.0
        -Real.0 = Real.0
        -Real.1 < Real.0
        lt_add_right(-Real.1, Real.0, p)
        -Real.1 + p < Real.0 + p
        -Real.1 + p = p + -Real.1
        p + -Real.1 = p - Real.1
        -Real.1 + p = p - Real.1
        Real.0 + p = p
        p - Real.1 < p
        div_lt_div_pos(p - Real.1, p, p)
        (p - Real.1) / p < p / p
        mul_div_cancel(Real.1, p)
        p * (Real.1 / p) = Real.1
        p / p = Real.1
        (p - Real.1) / p < Real.1
        Real.1 / conjugate_exponent(p) < Real.1
        conjugate_exponent(p) > Real.0
        real_inverse_antitone_pos_strict(Real.1 / conjugate_exponent(p), Real.1)
        Real.1.inverse < (Real.1 / conjugate_exponent(p)).inverse
        (Real.1 / conjugate_exponent(p)).inverse = conjugate_exponent(p)
        Real.1.inverse = Real.1
        Real.1 < conjugate_exponent(p)
        conjugate_exponent(p) > Real.1
    }
}

/// The conjugate exponents multiply: (p - 1) q = p.
theorem conjugate_exponent_pow(p: Real) {
    p > Real.1 implies (p - Real.1) * conjugate_exponent(p) = p
} by {
    if p > Real.1 {
        conjugate_exponent(p) = p / (p - Real.1)
        (p - Real.1) * conjugate_exponent(p) = (p - Real.1) * (p / (p - Real.1))
        (p - Real.1) * (p / (p - Real.1)) = (p - Real.1) * (p * (Real.1 / (p - Real.1)))
        (p - Real.1) * (p * (Real.1 / (p - Real.1))) = p * ((p - Real.1) * (Real.1 / (p - Real.1)))
        p - Real.1 != Real.0
        mul_div_cancel(Real.1, p - Real.1)
        (p - Real.1) * (Real.1 / (p - Real.1)) = Real.1
        p * ((p - Real.1) * (Real.1 / (p - Real.1))) = p * Real.1
        p * Real.1 = p
        (p - Real.1) * conjugate_exponent(p) = p
    }
}

/// A nonnegative real is its own absolute value.
theorem abs_eq_self_of_nonneg(x: Real) {
    x >= Real.0 implies x.abs = x
} by {
    if x >= Real.0 {
        if x.is_negative {
            neg_lt_zero(x)
            x < Real.0
            lte_lt_trans(Real.0, x, Real.0)
            Real.0 < Real.0
            false
        }
        x.abs = x
    }
}

/// The absolute value of a product with a nonnegative factor.
theorem abs_mul_nonneg_val(x: Real, y: Real) {
    y >= Real.0 implies (x * y).abs = x.abs * y
} by {
    if y >= Real.0 {
        mul_abs(x, y)
        x.abs * y.abs = (x * y).abs
        abs_eq_self_of_nonneg(y)
        y.abs = y
        x.abs * y = (x * y).abs
        (x * y).abs = x.abs * y
    }
}

/// A power value of a nonnegative base is nonnegative.
theorem rpow_value_nonneg(base: Real, exponent: Real, value: Real) {
    base >= Real.0 and exponent > Real.0 and base.rpow(exponent) = Option.some(value)
    implies value >= Real.0
} by {
    if base >= Real.0 and exponent > Real.0 and base.rpow(exponent) = Option.some(value) {
        rpow_nonneg_base_value(base, exponent)
        let (v: Real) satisfy {
            base.rpow(exponent) = Option.some(v) and v >= Real.0
        }
        some_injective[Real](value, v)
        value = v
        v >= Real.0
        value >= Real.0
    }
}

/// A power value of a positive base is positive.
theorem rpow_value_pos(base: Real, exponent: Real, value: Real) {
    base > Real.0 and exponent > Real.0 and base.rpow(exponent) = Option.some(value)
    implies value > Real.0
} by {
    if base > Real.0 and exponent > Real.0 and base.rpow(exponent) = Option.some(value) {
        rpow_pos(base, exponent)
        let (v: Real) satisfy {
            base.rpow(exponent) = Option.some(v) and v > Real.0
        }
        some_injective[Real](value, v)
        value = v
        v > Real.0
        value > Real.0
    }
}

/// The sequence |a_i + b_i|^(p - 1).
define abs_sum_pow_minus_one(a: Nat -> Real, b: Nat -> Real, p: Real, i: Nat) -> Real {
    abs_pow_value(add_fn(a, b), p - Real.1, i)
}

/// The pointwise product |a_i| · |a_i + b_i|^(p - 1).
define abs_mul_conjugate(a: Nat -> Real, b: Nat -> Real, p: Real, i: Nat) -> Real {
    a(i).abs * abs_sum_pow_minus_one(a, b, p, i)
}

/// The pointwise triangle-expanded product (|a_i| + |b_i|) · |a_i + b_i|^(p - 1).
define abs_triangle_expand(a: Nat -> Real, b: Nat -> Real, p: Real, i: Nat) -> Real {
    (a(i).abs + b(i).abs) * abs_sum_pow_minus_one(a, b, p, i)
}

/// The pointwise triangle-bound term |a_i + b_i| · |a_i + b_i|^(p - 1).
define abs_sum_triangle_term(a: Nat -> Real, b: Nat -> Real, p: Real, i: Nat) -> Real {
    (a(i) + b(i)).abs * abs_sum_pow_minus_one(a, b, p, i)
}

/// The p-th power of an absolute value splits off one factor:
/// |x|^p = |x| · |x|^(p - 1).
theorem abs_pow_value_split(a: Nat -> Real, p: Real, i: Nat) {
    p > Real.1 implies abs_pow_value(a, p, i) = a(i).abs * abs_pow_value(a, p - Real.1, i)
} by {
    if p > Real.1 {
        sub_one_pos(p)
        p - Real.1 > Real.0
        if a(i).abs > Real.0 {
            abs_pow_value(a, p, i) = (p * (a(i).abs).log.get_or_else(Real.0)).exp
            abs_pow_value(a, p - Real.1, i) = ((p - Real.1) * (a(i).abs).log.get_or_else(Real.0)).exp
            p - Real.1 = p + -Real.1
            (p - Real.1) + Real.1 = (p + -Real.1) + Real.1
            (p + -Real.1) + Real.1 = p + (-Real.1 + Real.1)
            add_neg_eq_zero(Real.1)
            Real.1 + -Real.1 = Real.0
            -Real.1 + Real.1 = Real.0
            p + (-Real.1 + Real.1) = p + Real.0
            add_zero_right(p)
            p + Real.0 = p
            (p - Real.1) + Real.1 = p
            Real.1 + (p - Real.1) = p
            (Real.1 + (p - Real.1)) * (a(i).abs).log.get_or_else(Real.0) = p * (a(i).abs).log.get_or_else(Real.0)
            mul_distrib_right(Real.1, p - Real.1, (a(i).abs).log.get_or_else(Real.0))
            (Real.1 + (p - Real.1)) * (a(i).abs).log.get_or_else(Real.0) = Real.1 * (a(i).abs).log.get_or_else(Real.0) + (p - Real.1) * (a(i).abs).log.get_or_else(Real.0)
            p * (a(i).abs).log.get_or_else(Real.0) = Real.1 * (a(i).abs).log.get_or_else(Real.0) + (p - Real.1) * (a(i).abs).log.get_or_else(Real.0)
            (p * (a(i).abs).log.get_or_else(Real.0)).exp = (Real.1 * (a(i).abs).log.get_or_else(Real.0) + (p - Real.1) * (a(i).abs).log.get_or_else(Real.0)).exp
            exp_add(Real.1 * (a(i).abs).log.get_or_else(Real.0), (p - Real.1) * (a(i).abs).log.get_or_else(Real.0))
            (Real.1 * (a(i).abs).log.get_or_else(Real.0) + (p - Real.1) * (a(i).abs).log.get_or_else(Real.0)).exp = (Real.1 * (a(i).abs).log.get_or_else(Real.0)).exp * ((p - Real.1) * (a(i).abs).log.get_or_else(Real.0)).exp
            mul_one_left((a(i).abs).log.get_or_else(Real.0))
            Real.1 * (a(i).abs).log.get_or_else(Real.0) = (a(i).abs).log.get_or_else(Real.0)
            (Real.1 * (a(i).abs).log.get_or_else(Real.0)).exp = ((a(i).abs).log.get_or_else(Real.0)).exp
            log_some_of_pos_exists(a(i).abs)
            let la: Real satisfy {
                a(i).abs.log = Option.some(la)
            }
            (a(i).abs).log.get_or_else(Real.0) = la
            exp_log_or_zero(a(i).abs, la)
            la.exp = a(i).abs
            ((a(i).abs).log.get_or_else(Real.0)).exp = a(i).abs
            (Real.1 * (a(i).abs).log.get_or_else(Real.0)).exp = a(i).abs
            (p * (a(i).abs).log.get_or_else(Real.0)).exp = a(i).abs * ((p - Real.1) * (a(i).abs).log.get_or_else(Real.0)).exp
            abs_pow_value(a, p, i) = a(i).abs * abs_pow_value(a, p - Real.1, i)
        } else {
            a(i).abs = Real.0
            abs_pow_value_of_abs_zero(a, p, i)
            abs_pow_value(a, p, i) = Real.0
            abs_pow_value_of_abs_zero(a, p - Real.1, i)
            abs_pow_value(a, p - Real.1, i) = Real.0
            a(i).abs * abs_pow_value(a, p - Real.1, i) = Real.0
            abs_pow_value(a, p, i) = a(i).abs * abs_pow_value(a, p - Real.1, i)
        }
    }
}

/// The triangle bound holds pointwise for the expanded terms.
theorem abs_triangle_term_le(a: Nat -> Real, b: Nat -> Real, p: Real, i: Nat) {
    p > Real.1 implies abs_sum_triangle_term(a, b, p, i) <= abs_triangle_expand(a, b, p, i)
} by {
    if p > Real.1 {
        sub_one_pos(p)
        p - Real.1 > Real.0
        abs_pow_value_nonneg(add_fn(a, b), p - Real.1, i)
        abs_sum_pow_minus_one(a, b, p, i) >= Real.0
        triangle_ineq(a(i), b(i))
        (a(i) + b(i)).abs <= a(i).abs + b(i).abs
        lte_mul_nonneg_right((a(i) + b(i)).abs, a(i).abs + b(i).abs, abs_sum_pow_minus_one(a, b, p, i))
        (a(i) + b(i)).abs * abs_sum_pow_minus_one(a, b, p, i) <= (a(i).abs + b(i).abs) * abs_sum_pow_minus_one(a, b, p, i)
        abs_sum_triangle_term(a, b, p, i) = (a(i) + b(i)).abs * abs_sum_pow_minus_one(a, b, p, i)
        abs_triangle_expand(a, b, p, i) = (a(i).abs + b(i).abs) * abs_sum_pow_minus_one(a, b, p, i)
        abs_sum_triangle_term(a, b, p, i) <= abs_triangle_expand(a, b, p, i)
    }
}

/// The expanded triangle product splits into two conjugate products.
theorem abs_triangle_expand_add(a: Nat -> Real, b: Nat -> Real, p: Real, i: Nat) {
    abs_triangle_expand(a, b, p, i) = abs_mul_conjugate(a, b, p, i) + abs_mul_conjugate(b, a, p, i)
} by {
    abs_triangle_expand(a, b, p, i) = (a(i).abs + b(i).abs) * abs_sum_pow_minus_one(a, b, p, i)
    mul_distrib_right(a(i).abs, b(i).abs, abs_sum_pow_minus_one(a, b, p, i))
    (a(i).abs + b(i).abs) * abs_sum_pow_minus_one(a, b, p, i) = a(i).abs * abs_sum_pow_minus_one(a, b, p, i) + b(i).abs * abs_sum_pow_minus_one(a, b, p, i)
    abs_mul_conjugate(a, b, p, i) = a(i).abs * abs_sum_pow_minus_one(a, b, p, i)
    add_fn(a, b, i) = a(i) + b(i)
    add_fn(b, a, i) = b(i) + a(i)
    a(i) + b(i) = b(i) + a(i)
    add_fn(a, b, i) = add_fn(b, a, i)
    abs_pow_value(add_fn(a, b), p - Real.1, i) = abs_pow_value(add_fn(b, a), p - Real.1, i)
    abs_sum_pow_minus_one(a, b, p, i) = abs_sum_pow_minus_one(b, a, p, i)
    abs_mul_conjugate(b, a, p, i) = b(i).abs * abs_sum_pow_minus_one(b, a, p, i)
    abs_mul_conjugate(b, a, p, i) = b(i).abs * abs_sum_pow_minus_one(a, b, p, i)
    abs_triangle_expand(a, b, p, i) = abs_mul_conjugate(a, b, p, i) + abs_mul_conjugate(b, a, p, i)
}

/// The product |a_i · c_i| with the conjugate-power sequence equals |a_i| · c_i.
theorem abs_product_conjugate_eq(a: Nat -> Real, b: Nat -> Real, p: Real, i: Nat) {
    p > Real.1 implies abs_product(a, abs_sum_pow_minus_one(a, b, p), i) = abs_mul_conjugate(a, b, p, i)
} by {
    if p > Real.1 {
        sub_one_pos(p)
        p - Real.1 > Real.0
        abs_pow_value_nonneg(add_fn(a, b), p - Real.1, i)
        abs_sum_pow_minus_one(a, b, p, i) >= Real.0
        abs_mul_nonneg_val(a(i), abs_sum_pow_minus_one(a, b, p, i))
        (a(i) * abs_sum_pow_minus_one(a, b, p, i)).abs = a(i).abs * abs_sum_pow_minus_one(a, b, p, i)
        abs_product(a, abs_sum_pow_minus_one(a, b, p), i) = (a(i) * abs_sum_pow_minus_one(a, b, p, i)).abs
        abs_mul_conjugate(a, b, p, i) = a(i).abs * abs_sum_pow_minus_one(a, b, p, i)
        abs_product(a, abs_sum_pow_minus_one(a, b, p), i) = abs_mul_conjugate(a, b, p, i)
    }
}

/// The q-th power of the (p-1)-th power of |a_i + b_i| is the p-th power.
theorem abs_pow_value_pow_pow(a: Nat -> Real, b: Nat -> Real, p: Real, q: Real, i: Nat) {
    p > Real.1 and q > Real.0 and (p - Real.1) * q = p
    implies abs_pow_value(abs_sum_pow_minus_one(a, b, p), q, i) = abs_pow_value(add_fn(a, b), p, i)
} by {
    if p > Real.1 and q > Real.0 and (p - Real.1) * q = p {
        sub_one_pos(p)
        p - Real.1 > Real.0
        abs_pow_value_nonneg(add_fn(a, b), p - Real.1, i)
        abs_sum_pow_minus_one(a, b, p, i) >= Real.0
        abs_eq_self_of_nonneg(abs_sum_pow_minus_one(a, b, p, i))
        abs_sum_pow_minus_one(a, b, p, i).abs = abs_sum_pow_minus_one(a, b, p, i)
        if (a(i) + b(i)).abs > Real.0 {
            add_fn(a, b, i) = a(i) + b(i)
            (add_fn(a, b, i)).abs = (a(i) + b(i)).abs
            (add_fn(a, b, i)).abs > Real.0
            abs_pow_value(add_fn(a, b), p - Real.1, i) = ((p - Real.1) * ((add_fn(a, b, i)).abs).log.get_or_else(Real.0)).exp
            ((add_fn(a, b, i)).abs).log.get_or_else(Real.0) = ((a(i) + b(i)).abs).log.get_or_else(Real.0)
            ((p - Real.1) * ((add_fn(a, b, i)).abs).log.get_or_else(Real.0)).exp = ((p - Real.1) * ((a(i) + b(i)).abs).log.get_or_else(Real.0)).exp
            abs_pow_value(add_fn(a, b), p - Real.1, i) = ((p - Real.1) * ((a(i) + b(i)).abs).log.get_or_else(Real.0)).exp
            abs_sum_pow_minus_one(a, b, p, i) = ((p - Real.1) * ((a(i) + b(i)).abs).log.get_or_else(Real.0)).exp
            exp_pos((p - Real.1) * ((a(i) + b(i)).abs).log.get_or_else(Real.0))
            ((p - Real.1) * ((a(i) + b(i)).abs).log.get_or_else(Real.0)).exp > Real.0
            abs_sum_pow_minus_one(a, b, p, i) > Real.0
            abs_sum_pow_minus_one(a, b, p, i).abs > Real.0
            abs_pow_value(abs_sum_pow_minus_one(a, b, p), q, i) = (q * (abs_sum_pow_minus_one(a, b, p, i)).log.get_or_else(Real.0)).exp
            (abs_sum_pow_minus_one(a, b, p, i)).log.get_or_else(Real.0) = (((p - Real.1) * ((a(i) + b(i)).abs).log.get_or_else(Real.0)).exp).log.get_or_else(Real.0)
            log_value_exp((p - Real.1) * ((a(i) + b(i)).abs).log.get_or_else(Real.0))
            (((p - Real.1) * ((a(i) + b(i)).abs).log.get_or_else(Real.0)).exp).log.get_or_else(Real.0) = (p - Real.1) * ((a(i) + b(i)).abs).log.get_or_else(Real.0)
            (abs_sum_pow_minus_one(a, b, p, i)).log.get_or_else(Real.0) = (p - Real.1) * ((a(i) + b(i)).abs).log.get_or_else(Real.0)
            q * (abs_sum_pow_minus_one(a, b, p, i)).log.get_or_else(Real.0) = q * ((p - Real.1) * ((a(i) + b(i)).abs).log.get_or_else(Real.0))
            q * ((p - Real.1) * ((a(i) + b(i)).abs).log.get_or_else(Real.0)) = (q * (p - Real.1)) * ((a(i) + b(i)).abs).log.get_or_else(Real.0)
            q * (p - Real.1) = (p - Real.1) * q
            (p - Real.1) * q = p
            q * (p - Real.1) = p
            (q * (p - Real.1)) * ((a(i) + b(i)).abs).log.get_or_else(Real.0) = p * ((a(i) + b(i)).abs).log.get_or_else(Real.0)
            q * (abs_sum_pow_minus_one(a, b, p, i)).log.get_or_else(Real.0) = p * ((a(i) + b(i)).abs).log.get_or_else(Real.0)
            (q * (abs_sum_pow_minus_one(a, b, p, i)).log.get_or_else(Real.0)).exp = (p * ((a(i) + b(i)).abs).log.get_or_else(Real.0)).exp
            (add_fn(a, b, i)).abs = (a(i) + b(i)).abs
            (add_fn(a, b, i)).abs > Real.0
            abs_pow_value(add_fn(a, b), p, i) = (p * ((add_fn(a, b, i)).abs).log.get_or_else(Real.0)).exp
            ((add_fn(a, b, i)).abs).log.get_or_else(Real.0) = ((a(i) + b(i)).abs).log.get_or_else(Real.0)
            (p * ((add_fn(a, b, i)).abs).log.get_or_else(Real.0)).exp = (p * ((a(i) + b(i)).abs).log.get_or_else(Real.0)).exp
            abs_pow_value(add_fn(a, b), p, i) = (p * ((a(i) + b(i)).abs).log.get_or_else(Real.0)).exp
            (q * (abs_sum_pow_minus_one(a, b, p, i)).log.get_or_else(Real.0)).exp = abs_pow_value(add_fn(a, b), p, i)
            abs_pow_value(abs_sum_pow_minus_one(a, b, p), q, i) = abs_pow_value(add_fn(a, b), p, i)
        } else {
            not (a(i) + b(i)).abs > Real.0
            not_gt_imp_lte((a(i) + b(i)).abs, Real.0)
            (a(i) + b(i)).abs <= Real.0
            abs_gte_zero(a(i) + b(i))
            (a(i) + b(i)).abs >= Real.0
            lte_antisymm((a(i) + b(i)).abs, Real.0)
            (a(i) + b(i)).abs = Real.0
            add_fn(a, b, i) = a(i) + b(i)
            (add_fn(a, b, i)).abs = (a(i) + b(i)).abs
            (add_fn(a, b, i)).abs = Real.0
            abs_pow_value_of_abs_zero(add_fn(a, b), p, i)
            abs_pow_value(add_fn(a, b), p, i) = Real.0
            abs_pow_value_of_abs_zero(add_fn(a, b), p - Real.1, i)
            abs_sum_pow_minus_one(a, b, p, i) = Real.0
            abs_pow_value(abs_sum_pow_minus_one(a, b, p), q, i) = Real.0
            abs_pow_value(abs_sum_pow_minus_one(a, b, p), q, i) = abs_pow_value(add_fn(a, b), p, i)
        }
    }
}

/// The q-norm power sum of the conjugate sequence equals the p-norm power sum.
theorem abs_pow_sum_c_eq_w(a: Nat -> Real, b: Nat -> Real, n: Nat, p: Real, q: Real) {
    p > Real.1 and q > Real.0 and (p - Real.1) * q = p
    implies abs_pow_sum(abs_sum_pow_minus_one(a, b, p), q, n) = abs_pow_sum(add_fn(a, b), p, n)
} by {
    if p > Real.1 and q > Real.0 and (p - Real.1) * q = p {
        forall(i: Nat) {
            if i < n {
                abs_pow_value_pow_pow(a, b, p, q, i)
                abs_pow_value(abs_sum_pow_minus_one(a, b, p), q, i) = abs_pow_value(add_fn(a, b), p, i)
            }
        }
        partial_pointwise_eq(abs_pow_value(abs_sum_pow_minus_one(a, b, p), q), abs_pow_value(add_fn(a, b), p), n)
        partial(abs_pow_value(abs_sum_pow_minus_one(a, b, p), q), n) = partial(abs_pow_value(add_fn(a, b), p), n)
        abs_pow_sum(abs_sum_pow_minus_one(a, b, p), q, n) = partial(abs_pow_value(abs_sum_pow_minus_one(a, b, p), q), n)
        abs_pow_sum(add_fn(a, b), p, n) = partial(abs_pow_value(add_fn(a, b), p), n)
        abs_pow_sum(abs_sum_pow_minus_one(a, b, p), q, n) = abs_pow_sum(add_fn(a, b), p, n)
    }
}

/// The Hölder left-hand side with the conjugate sequence equals the sum of
/// |a_i| c_i.
theorem holder_lhs_conjugate_sum(a: Nat -> Real, b: Nat -> Real, n: Nat, p: Real) {
    p > Real.1 implies
        partial(abs_product(a, abs_sum_pow_minus_one(a, b, p)), n) = partial(abs_mul_conjugate(a, b, p), n)
} by {
    if p > Real.1 {
        forall(i: Nat) {
            if i < n {
                abs_product_conjugate_eq(a, b, p, i)
                abs_product(a, abs_sum_pow_minus_one(a, b, p), i) = abs_mul_conjugate(a, b, p, i)
            }
        }
        partial_pointwise_eq(abs_product(a, abs_sum_pow_minus_one(a, b, p)), abs_mul_conjugate(a, b, p), n)
        partial(abs_product(a, abs_sum_pow_minus_one(a, b, p)), n) = partial(abs_mul_conjugate(a, b, p), n)
    }
}

/// Minkowski's inequality for finite real sequences, value form: for p > 1
/// with uu, vv, ww the p-norms of a, b, a + b and ww_q the conjugate-norm
/// of a + b, the p-norm of a + b is at most the sum of the p-norms of a
/// and b.
theorem finite_minkowski(a: Nat -> Real, b: Nat -> Real, n: Nat, p: Real, uu: Real, vv: Real, ww: Real, ww_q: Real) {
    p > Real.1
    and (abs_pow_sum(a, p, n)).rpow(Real.1 / p) = Option.some(uu)
    and (abs_pow_sum(b, p, n)).rpow(Real.1 / p) = Option.some(vv)
    and (abs_pow_sum(add_fn(a, b), p, n)).rpow(Real.1 / p) = Option.some(ww)
    and (abs_pow_sum(add_fn(a, b), p, n)).rpow(Real.1 / conjugate_exponent(p)) = Option.some(ww_q)
    implies ww <= uu + vv
} by {
    if p > Real.1
        and (abs_pow_sum(a, p, n)).rpow(Real.1 / p) = Option.some(uu)
        and (abs_pow_sum(b, p, n)).rpow(Real.1 / p) = Option.some(vv)
        and (abs_pow_sum(add_fn(a, b), p, n)).rpow(Real.1 / p) = Option.some(ww)
        and (abs_pow_sum(add_fn(a, b), p, n)).rpow(Real.1 / conjugate_exponent(p)) = Option.some(ww_q) {
        gt_one_imp_pos(p)
        p > Real.0
        if abs_pow_sum(add_fn(a, b), p, n) > Real.0 {
            rpow_value_pos(abs_pow_sum(add_fn(a, b), p, n), Real.1 / p, ww)
            ww > Real.0
            conjugate_exponent_gt_one(p)
            conjugate_exponent(p) > Real.1
            gt_one_imp_pos(conjugate_exponent(p))
            conjugate_exponent(p) > Real.0
            real_one_div_pos(conjugate_exponent(p))
            Real.1 / conjugate_exponent(p) > Real.0
            rpow_value_pos(abs_pow_sum(add_fn(a, b), p, n), Real.1 / conjugate_exponent(p), ww_q)
            ww_q > Real.0
            forall(i: Nat) {
                if i < n {
                    abs_pow_value_split(add_fn(a, b), p, i)
                    abs_pow_value(add_fn(a, b), p, i) = (a(i) + b(i)).abs * abs_pow_value(add_fn(a, b), p - Real.1, i)
                    add_fn(a, b, i) = a(i) + b(i)
                    abs_sum_pow_minus_one(a, b, p, i) = abs_pow_value(add_fn(a, b), p - Real.1, i)
                    abs_sum_triangle_term(a, b, p, i) = (a(i) + b(i)).abs * abs_sum_pow_minus_one(a, b, p, i)
                    abs_sum_triangle_term(a, b, p, i) = (a(i) + b(i)).abs * abs_pow_value(add_fn(a, b), p - Real.1, i)
                    abs_pow_value(add_fn(a, b), p, i) = abs_sum_triangle_term(a, b, p, i)
                }
            }
            partial_pointwise_eq(abs_pow_value(add_fn(a, b), p), abs_sum_triangle_term(a, b, p), n)
            partial(abs_pow_value(add_fn(a, b), p), n) = partial(abs_sum_triangle_term(a, b, p), n)
            abs_pow_sum(add_fn(a, b), p, n) = partial(abs_pow_value(add_fn(a, b), p), n)
            abs_pow_sum(add_fn(a, b), p, n) = partial(abs_sum_triangle_term(a, b, p), n)
            forall(i: Nat) {
                if i < n {
                    abs_triangle_term_le(a, b, p, i)
                    abs_sum_triangle_term(a, b, p, i) <= abs_triangle_expand(a, b, p, i)
                }
            }
            partial_le_range(abs_sum_triangle_term(a, b, p), abs_triangle_expand(a, b, p), n)
            partial(abs_sum_triangle_term(a, b, p), n) <= partial(abs_triangle_expand(a, b, p), n)
            abs_pow_sum(add_fn(a, b), p, n) <= partial(abs_triangle_expand(a, b, p), n)
            forall(i: Nat) {
                if i < n {
                    abs_triangle_expand_add(a, b, p, i)
                    abs_triangle_expand(a, b, p, i) = abs_mul_conjugate(a, b, p, i) + abs_mul_conjugate(b, a, p, i)
                    add_fn(abs_mul_conjugate(a, b, p), abs_mul_conjugate(b, a, p), i) = abs_mul_conjugate(a, b, p, i) + abs_mul_conjugate(b, a, p, i)
                    abs_triangle_expand(a, b, p, i) = add_fn(abs_mul_conjugate(a, b, p), abs_mul_conjugate(b, a, p), i)
                }
            }
            partial_pointwise_eq(abs_triangle_expand(a, b, p), add_fn(abs_mul_conjugate(a, b, p), abs_mul_conjugate(b, a, p)), n)
            partial(abs_triangle_expand(a, b, p), n) = partial(add_fn(abs_mul_conjugate(a, b, p), abs_mul_conjugate(b, a, p)), n)
            partial_add(abs_mul_conjugate(a, b, p), abs_mul_conjugate(b, a, p), n)
            partial(abs_mul_conjugate(a, b, p), n) + partial(abs_mul_conjugate(b, a, p), n) = partial(add_fn(abs_mul_conjugate(a, b, p), abs_mul_conjugate(b, a, p)), n)
            partial(abs_triangle_expand(a, b, p), n) = partial(abs_mul_conjugate(a, b, p), n) + partial(abs_mul_conjugate(b, a, p), n)
            abs_pow_sum(add_fn(a, b), p, n) <= partial(abs_mul_conjugate(a, b, p), n) + partial(abs_mul_conjugate(b, a, p), n)
            conjugate_exponent_recip(p)
            Real.1 / p + Real.1 / conjugate_exponent(p) = Real.1
            conjugate_exponent_pow(p)
            (p - Real.1) * conjugate_exponent(p) = p
            abs_pow_sum_c_eq_w(a, b, n, p, conjugate_exponent(p))
            abs_pow_sum(abs_sum_pow_minus_one(a, b, p), conjugate_exponent(p), n) = abs_pow_sum(add_fn(a, b), p, n)
            (abs_pow_sum(abs_sum_pow_minus_one(a, b, p), conjugate_exponent(p), n)).rpow(Real.1 / conjugate_exponent(p)) = (abs_pow_sum(add_fn(a, b), p, n)).rpow(Real.1 / conjugate_exponent(p))
            (abs_pow_sum(abs_sum_pow_minus_one(a, b, p), conjugate_exponent(p), n)).rpow(Real.1 / conjugate_exponent(p)) = Option.some(ww_q)
            finite_holder(a, abs_sum_pow_minus_one(a, b, p), n, p, conjugate_exponent(p), uu, ww_q)
            partial(abs_product(a, abs_sum_pow_minus_one(a, b, p)), n) <= uu * ww_q
            holder_lhs_conjugate_sum(a, b, n, p)
            partial(abs_product(a, abs_sum_pow_minus_one(a, b, p)), n) = partial(abs_mul_conjugate(a, b, p), n)
            partial(abs_mul_conjugate(a, b, p), n) <= uu * ww_q
            finite_holder(b, abs_sum_pow_minus_one(a, b, p), n, p, conjugate_exponent(p), vv, ww_q)
            partial(abs_product(b, abs_sum_pow_minus_one(a, b, p)), n) <= vv * ww_q
            forall(i: Nat) {
                if i < n {
                    abs_product_conjugate_eq(b, a, p, i)
                    abs_product(b, abs_sum_pow_minus_one(b, a, p), i) = abs_mul_conjugate(b, a, p, i)
                    add_fn(b, a, i) = b(i) + a(i)
                    add_fn(a, b, i) = a(i) + b(i)
                    a(i) + b(i) = b(i) + a(i)
                    add_fn(a, b, i) = add_fn(b, a, i)
                    abs_pow_value(add_fn(a, b), p - Real.1, i) = abs_pow_value(add_fn(b, a), p - Real.1, i)
                    abs_sum_pow_minus_one(a, b, p, i) = abs_sum_pow_minus_one(b, a, p, i)
                    abs_product(b, abs_sum_pow_minus_one(a, b, p), i) = abs_product(b, abs_sum_pow_minus_one(b, a, p), i)
                    abs_product(b, abs_sum_pow_minus_one(a, b, p), i) = abs_mul_conjugate(b, a, p, i)
                }
            }
            partial_pointwise_eq(abs_product(b, abs_sum_pow_minus_one(a, b, p)), abs_mul_conjugate(b, a, p), n)
            partial(abs_product(b, abs_sum_pow_minus_one(a, b, p)), n) = partial(abs_mul_conjugate(b, a, p), n)
            partial(abs_mul_conjugate(b, a, p), n) <= vv * ww_q
            add_lte_add(partial(abs_mul_conjugate(a, b, p), n), uu * ww_q, partial(abs_mul_conjugate(b, a, p), n), vv * ww_q)
            partial(abs_mul_conjugate(a, b, p), n) + partial(abs_mul_conjugate(b, a, p), n) <= uu * ww_q + vv * ww_q
            lte_trans(abs_pow_sum(add_fn(a, b), p, n), partial(abs_mul_conjugate(a, b, p), n) + partial(abs_mul_conjugate(b, a, p), n), uu * ww_q + vv * ww_q)
            abs_pow_sum(add_fn(a, b), p, n) <= uu * ww_q + vv * ww_q
            mul_distrib_right(uu, vv, ww_q)
            (uu + vv) * ww_q = uu * ww_q + vv * ww_q
            uu * ww_q + vv * ww_q = (uu + vv) * ww_q
            abs_pow_sum(add_fn(a, b), p, n) <= (uu + vv) * ww_q
            ww_q > Real.0
            real_one_div_pos(ww_q)
            Real.1 / ww_q > Real.0
            mul_le_mul_pos_right(abs_pow_sum(add_fn(a, b), p, n), (uu + vv) * ww_q, Real.1 / ww_q)
            abs_pow_sum(add_fn(a, b), p, n) * (Real.1 / ww_q) <= ((uu + vv) * ww_q) * (Real.1 / ww_q)
            Real.1 / p + Real.1 / conjugate_exponent(p) = Real.1
            Real.1 / conjugate_exponent(p) + Real.1 / p = Real.1
            rpow_one(abs_pow_sum(add_fn(a, b), p, n))
            (abs_pow_sum(add_fn(a, b), p, n)).rpow(Real.1) = Option.some(abs_pow_sum(add_fn(a, b), p, n))
            (abs_pow_sum(add_fn(a, b), p, n)).rpow(Real.1 / conjugate_exponent(p) + Real.1 / p) = Option.some(abs_pow_sum(add_fn(a, b), p, n))
            rpow_mul_val(abs_pow_sum(add_fn(a, b), p, n), Real.1 / conjugate_exponent(p), Real.1 / p, ww_q, ww, abs_pow_sum(add_fn(a, b), p, n))
            abs_pow_sum(add_fn(a, b), p, n) = ww_q * ww
            abs_pow_sum(add_fn(a, b), p, n) * (Real.1 / ww_q) = (ww_q * ww) * (Real.1 / ww_q)
            (ww_q * ww) * (Real.1 / ww_q) = ww * (ww_q * (Real.1 / ww_q))
            ww_q != Real.0
            mul_div_cancel(Real.1, ww_q)
            ww_q * (Real.1 / ww_q) = Real.1
            ww * (ww_q * (Real.1 / ww_q)) = ww * Real.1
            ww * Real.1 = ww
            abs_pow_sum(add_fn(a, b), p, n) * (Real.1 / ww_q) = ww
            ((uu + vv) * ww_q) * (Real.1 / ww_q) = (uu + vv) * (ww_q * (Real.1 / ww_q))
            (uu + vv) * (ww_q * (Real.1 / ww_q)) = (uu + vv) * Real.1
            (uu + vv) * Real.1 = uu + vv
            ((uu + vv) * ww_q) * (Real.1 / ww_q) = uu + vv
            ww <= uu + vv
        } else {
            not abs_pow_sum(add_fn(a, b), p, n) > Real.0
            not_gt_imp_lte(abs_pow_sum(add_fn(a, b), p, n), Real.0)
            abs_pow_sum(add_fn(a, b), p, n) <= Real.0
            abs_pow_sum_nonneg(add_fn(a, b), p, n)
            abs_pow_sum(add_fn(a, b), p, n) >= Real.0
            lte_antisymm(abs_pow_sum(add_fn(a, b), p, n), Real.0)
            abs_pow_sum(add_fn(a, b), p, n) = Real.0
            rpow_root_zero_of_sum_zero(add_fn(a, b), n, p, ww)
            ww = Real.0
            real_one_div_pos(p)
            Real.1 / p > Real.0
            abs_pow_sum_nonneg(a, p, n)
            abs_pow_sum(a, p, n) >= Real.0
            rpow_value_nonneg(abs_pow_sum(a, p, n), Real.1 / p, uu)
            uu >= Real.0
            abs_pow_sum_nonneg(b, p, n)
            abs_pow_sum(b, p, n) >= Real.0
            rpow_value_nonneg(abs_pow_sum(b, p, n), Real.1 / p, vv)
            vv >= Real.0
            lte_add_right(Real.0, uu, vv)
            Real.0 + vv <= uu + vv
            Real.0 + vv = vv
            vv <= uu + vv
            lte_trans(Real.0, vv, uu + vv)
            Real.0 <= uu + vv
            ww = Real.0
            ww <= uu + vv
        }
    }
}

/// Minkowski's inequality for finite real sequences: for p > 1, the p-norm
/// of a + b is at most the sum of the p-norms of a and b.
theorem finite_minkowski_exists(a: Nat -> Real, b: Nat -> Real, n: Nat, p: Real) {
    p > Real.1 implies exists(uu: Real, vv: Real, ww: Real) {
        (abs_pow_sum(a, p, n)).rpow(Real.1 / p) = Option.some(uu)
        and (abs_pow_sum(b, p, n)).rpow(Real.1 / p) = Option.some(vv)
        and (abs_pow_sum(add_fn(a, b), p, n)).rpow(Real.1 / p) = Option.some(ww)
        and ww <= uu + vv
    }
} by {
    if p > Real.1 {
        gt_one_imp_pos(p)
        p > Real.0
        abs_pow_sum_nonneg(a, p, n)
        abs_pow_sum(a, p, n) >= Real.0
        real_one_div_pos(p)
        Real.1 / p > Real.0
        rpow_nonneg_base_value(abs_pow_sum(a, p, n), Real.1 / p)
        let (u0: Real) satisfy {
            (abs_pow_sum(a, p, n)).rpow(Real.1 / p) = Option.some(u0) and u0 >= Real.0
        }
        abs_pow_sum_nonneg(b, p, n)
        abs_pow_sum(b, p, n) >= Real.0
        rpow_nonneg_base_value(abs_pow_sum(b, p, n), Real.1 / p)
        let (v0: Real) satisfy {
            (abs_pow_sum(b, p, n)).rpow(Real.1 / p) = Option.some(v0) and v0 >= Real.0
        }
        abs_pow_sum_nonneg(add_fn(a, b), p, n)
        abs_pow_sum(add_fn(a, b), p, n) >= Real.0
        rpow_nonneg_base_value(abs_pow_sum(add_fn(a, b), p, n), Real.1 / p)
        let (w0: Real) satisfy {
            (abs_pow_sum(add_fn(a, b), p, n)).rpow(Real.1 / p) = Option.some(w0) and w0 >= Real.0
        }
        conjugate_exponent_gt_one(p)
        conjugate_exponent(p) > Real.1
        gt_one_imp_pos(conjugate_exponent(p))
        conjugate_exponent(p) > Real.0
        real_one_div_pos(conjugate_exponent(p))
        Real.1 / conjugate_exponent(p) > Real.0
        rpow_nonneg_base_value(abs_pow_sum(add_fn(a, b), p, n), Real.1 / conjugate_exponent(p))
        let (wq0: Real) satisfy {
            (abs_pow_sum(add_fn(a, b), p, n)).rpow(Real.1 / conjugate_exponent(p)) = Option.some(wq0) and wq0 >= Real.0
        }
        finite_minkowski(a, b, n, p, u0, v0, w0, wq0)
        w0 <= u0 + v0
        exists(uu: Real, vv: Real, ww: Real) {
            (abs_pow_sum(a, p, n)).rpow(Real.1 / p) = Option.some(uu)
            and (abs_pow_sum(b, p, n)).rpow(Real.1 / p) = Option.some(vv)
            and (abs_pow_sum(add_fn(a, b), p, n)).rpow(Real.1 / p) = Option.some(ww)
            and ww <= uu + vv
        }
    }
}

// =====================================================================
// The harmonic mean and the AM-GM-HM chain
// =====================================================================

/// The reciprocal sequence of a positive sequence.
define recip_fn(f: Nat -> Real, i: Nat) -> Real {
    Real.1 / f(i)
}

/// The harmonic mean of the first `n` positive values of a sequence.
define finite_harmonic_mean(f: Nat -> Real, n: Nat) -> Real {
    from_nat[Real](n) / partial(recip_fn(f), n)
}

/// The reciprocal sequence of a positive sequence is positive.
theorem positive_on_recip(f: Nat -> Real, n: Nat) {
    positive_on(f, n) implies positive_on(recip_fn(f), n)
} by {
    if positive_on(f, n) {
        forall(i: Nat) {
            if i < n {
                positive_on(f, n) = forall(j: Nat) { j < n implies f(j) > Real.0 }
                f(i) > Real.0
                real_one_div_pos(f(i))
                Real.1 / f(i) > Real.0
                recip_fn(f, i) = Real.1 / f(i)
                recip_fn(f, i) > Real.0
            }
        }
    }
}

/// The product of reciprocals is the reciprocal of the product.
theorem finite_real_product_recip(f: Nat -> Real, n: Nat) {
    positive_on(f, n) implies finite_real_product(recip_fn(f), n) = Real.1 / finite_real_product(f, n)
} by {
    if positive_on(f, n) {
        define pred(k: Nat) -> Bool {
            positive_on(f, k) implies finite_real_product(recip_fn(f), k) = Real.1 / finite_real_product(f, k)
        }
        finite_real_product_zero(recip_fn(f))
        finite_real_product(recip_fn(f), Nat.0) = Real.1
        finite_real_product_zero(f)
        finite_real_product(f, Nat.0) = Real.1
        Real.1 / Real.1 = Real.1
        finite_real_product(recip_fn(f), Nat.0) = Real.1 / finite_real_product(f, Nat.0)
        pred(Nat.0)
        forall(k: Nat) {
            if pred(k) {
                if positive_on(f, k.suc) {
                    k < k.suc
                    k <= k.suc
                    positive_on_prefix(f, k.suc, k)
                    positive_on(f, k)
                    pred(k) = (positive_on(f, k) implies finite_real_product(recip_fn(f), k) = Real.1 / finite_real_product(f, k))
                    finite_real_product(recip_fn(f), k) = Real.1 / finite_real_product(f, k)
                    positive_on(f, k.suc) = forall(i: Nat) { i < k.suc implies f(i) > Real.0 }
                    f(k) > Real.0
                    real_one_div_pos(f(k))
                    Real.1 / f(k) > Real.0
                    recip_fn(f, k) = Real.1 / f(k)
                    finite_real_product_pos(f, k)
                    finite_real_product(f, k) > Real.0
                    finite_real_product_suc(recip_fn(f), k)
                    finite_real_product(recip_fn(f), k.suc) = finite_real_product(recip_fn(f), k) * recip_fn(f, k)
                    finite_real_product(recip_fn(f), k) * recip_fn(f, k) = (Real.1 / finite_real_product(f, k)) * (Real.1 / f(k))
                    finite_real_product(f, k) != Real.0
                    f(k) != Real.0
                    mul_div(Real.1, finite_real_product(f, k), Real.1, f(k))
                    (Real.1 / finite_real_product(f, k)) * (Real.1 / f(k)) = (Real.1 * Real.1) / (finite_real_product(f, k) * f(k))
                    Real.1 * Real.1 = Real.1
                    (Real.1 * Real.1) / (finite_real_product(f, k) * f(k)) = Real.1 / (finite_real_product(f, k) * f(k))
                    finite_real_product_suc(f, k)
                    finite_real_product(f, k.suc) = finite_real_product(f, k) * f(k)
                    Real.1 / (finite_real_product(f, k) * f(k)) = Real.1 / finite_real_product(f, k.suc)
                    finite_real_product(recip_fn(f), k.suc) = Real.1 / finite_real_product(f, k.suc)
                }
                pred(k.suc)
            }
        }
        pred(Nat.0) and forall(k: Nat) { pred(k) implies pred(k.suc) }
        alt_induction(pred)
        forall(k: Nat) { pred(k) }
        pred(n)
        pred(n) = (positive_on(f, n) implies finite_real_product(recip_fn(f), n) = Real.1 / finite_real_product(f, n))
        finite_real_product(recip_fn(f), n) = Real.1 / finite_real_product(f, n)
    }
}

/// The geometric mean of the reciprocals is the reciprocal of the geometric mean.
theorem recip_geometric_mean(f: Nat -> Real, n: Nat, g: Real) {
    n != Nat.0 and positive_on(f, n) and (finite_real_product(f, n)).rpow(Real.1 / from_nat[Real](n)) = Option.some(g)
    implies (finite_real_product(recip_fn(f), n)).rpow(Real.1 / from_nat[Real](n)) = Option.some(Real.1 / g)
} by {
    if n != Nat.0 and positive_on(f, n) and (finite_real_product(f, n)).rpow(Real.1 / from_nat[Real](n)) = Option.some(g) {
        finite_real_product_pos(f, n)
        finite_real_product(f, n) > Real.0
        finite_real_product_recip(f, n)
        finite_real_product(recip_fn(f), n) = Real.1 / finite_real_product(f, n)
        rpow_inv_base(finite_real_product(f, n), Real.1 / from_nat[Real](n), g)
        (Real.1 / finite_real_product(f, n)).rpow(Real.1 / from_nat[Real](n)) = Option.some(Real.1 / g)
        (finite_real_product(recip_fn(f), n)).rpow(Real.1 / from_nat[Real](n)) = Option.some(Real.1 / g)
    }
}

/// The harmonic mean is the reciprocal of the arithmetic mean of the reciprocals.
theorem harmonic_mean_eq_recip_am(f: Nat -> Real, n: Nat) {
    n != Nat.0 and positive_on(f, n) implies
        finite_harmonic_mean(f, n) = Real.1 / finite_real_mean(recip_fn(f), n)
} by {
    if n != Nat.0 and positive_on(f, n) {
        positive_on_recip(f, n)
        positive_on(recip_fn(f), n)
        partial_pos_bounded(recip_fn(f), n)
        partial(recip_fn(f), n) > Real.0
        from_nat_real_pos_of_ne_zero(n)
        from_nat[Real](n) > Real.0
        div_pos_of_pos_pos(partial(recip_fn(f), n), from_nat[Real](n))
        partial(recip_fn(f), n) / from_nat[Real](n) > Real.0
        finite_real_mean(recip_fn(f), n) = partial(recip_fn(f), n) / from_nat[Real](n)
        partial(recip_fn(f), n) != Real.0
        from_nat[Real](n) != Real.0
        div_by_fraction(Real.1, Real.1, partial(recip_fn(f), n), from_nat[Real](n))
        (Real.1 / Real.1) / (partial(recip_fn(f), n) / from_nat[Real](n)) = (Real.1 / Real.1) * (from_nat[Real](n) / partial(recip_fn(f), n))
        Real.1 / Real.1 = Real.1
        (Real.1 / Real.1) / (partial(recip_fn(f), n) / from_nat[Real](n)) = Real.1 / (partial(recip_fn(f), n) / from_nat[Real](n))
        (Real.1 / Real.1) * (from_nat[Real](n) / partial(recip_fn(f), n)) = from_nat[Real](n) / partial(recip_fn(f), n)
        Real.1 / (partial(recip_fn(f), n) / from_nat[Real](n)) = from_nat[Real](n) / partial(recip_fn(f), n)
        Real.1 / finite_real_mean(recip_fn(f), n) = from_nat[Real](n) / partial(recip_fn(f), n)
        finite_harmonic_mean(f, n) = from_nat[Real](n) / partial(recip_fn(f), n)
        finite_harmonic_mean(f, n) = Real.1 / finite_real_mean(recip_fn(f), n)
    }
}

/// The harmonic mean is at most the geometric mean for positive values.
theorem harmonic_mean_le_geometric_mean(f: Nat -> Real, n: Nat) {
    n != Nat.0 and positive_on(f, n) implies exists(g: Real) {
        (finite_real_product(f, n)).rpow(Real.1 / from_nat[Real](n)) = Option.some(g)
        and finite_harmonic_mean(f, n) <= g
    }
} by {
    if n != Nat.0 and positive_on(f, n) {
        positive_on_recip(f, n)
        positive_on(recip_fn(f), n)
        am_gm_pos(recip_fn(f), n)
        let (gp: Real) satisfy {
            (finite_real_product(recip_fn(f), n)).rpow(Real.1 / from_nat[Real](n)) = Option.some(gp)
            and gp <= finite_real_mean(recip_fn(f), n)
        }
        from_nat_real_pos_of_ne_zero(n)
        from_nat[Real](n) > Real.0
        real_one_div_pos(from_nat[Real](n))
        Real.1 / from_nat[Real](n) > Real.0
        finite_real_product_pos(f, n)
        finite_real_product(f, n) > Real.0
        rpow_pos(finite_real_product(f, n), Real.1 / from_nat[Real](n))
        let (g: Real) satisfy {
            (finite_real_product(f, n)).rpow(Real.1 / from_nat[Real](n)) = Option.some(g) and g > Real.0
        }
        recip_geometric_mean(f, n, g)
        (finite_real_product(recip_fn(f), n)).rpow(Real.1 / from_nat[Real](n)) = Option.some(Real.1 / g)
        some_injective[Real](gp, Real.1 / g)
        gp = Real.1 / g
        gp <= finite_real_mean(recip_fn(f), n)
        Real.1 / g <= finite_real_mean(recip_fn(f), n)
        g > Real.0
        real_one_div_pos(g)
        Real.1 / g > Real.0
        partial_pos_bounded(recip_fn(f), n)
        partial(recip_fn(f), n) > Real.0
        div_pos_of_pos_pos(partial(recip_fn(f), n), from_nat[Real](n))
        partial(recip_fn(f), n) / from_nat[Real](n) > Real.0
        finite_real_mean(recip_fn(f), n) = partial(recip_fn(f), n) / from_nat[Real](n)
        finite_real_mean(recip_fn(f), n) > Real.0
        real_recip_antitone_pos(Real.1 / g, finite_real_mean(recip_fn(f), n))
        Real.1 / finite_real_mean(recip_fn(f), n) <= Real.1 / (Real.1 / g)
        one_div_one_div(g)
        Real.1 / (Real.1 / g) = g
        Real.1 / finite_real_mean(recip_fn(f), n) <= g
        harmonic_mean_eq_recip_am(f, n)
        finite_harmonic_mean(f, n) = Real.1 / finite_real_mean(recip_fn(f), n)
        finite_harmonic_mean(f, n) <= g
        exists(g0: Real) {
            (finite_real_product(f, n)).rpow(Real.1 / from_nat[Real](n)) = Option.some(g0)
            and finite_harmonic_mean(f, n) <= g0
        }
    }
}

/// The harmonic mean is at most the arithmetic mean for positive values.
theorem harmonic_mean_le_arithmetic_mean(f: Nat -> Real, n: Nat) {
    n != Nat.0 and positive_on(f, n) implies exists(a: Real) {
        a = finite_real_mean(f, n)
        and finite_harmonic_mean(f, n) <= a
    }
} by {
    if n != Nat.0 and positive_on(f, n) {
        am_gm_pos(f, n)
        let (g: Real) satisfy {
            (finite_real_product(f, n)).rpow(Real.1 / from_nat[Real](n)) = Option.some(g)
            and g <= finite_real_mean(f, n)
        }
        harmonic_mean_le_geometric_mean(f, n)
        let (g2: Real) satisfy {
            (finite_real_product(f, n)).rpow(Real.1 / from_nat[Real](n)) = Option.some(g2)
            and finite_harmonic_mean(f, n) <= g2
        }
        some_injective[Real](g, g2)
        g = g2
        finite_harmonic_mean(f, n) <= g2
        finite_harmonic_mean(f, n) <= g
        g <= finite_real_mean(f, n)
        lte_trans(finite_harmonic_mean(f, n), g, finite_real_mean(f, n))
        finite_harmonic_mean(f, n) <= finite_real_mean(f, n)
        exists(a: Real) {
            a = finite_real_mean(f, n)
            and finite_harmonic_mean(f, n) <= a
        }
    }
}

/// Hölder's inequality at p = q = 2 is Cauchy-Schwarz for absolute values.
theorem finite_holder_p2(a: Nat -> Real, b: Nat -> Real, n: Nat) {
    exists(uu: Real, vv: Real) {
        (abs_pow_sum(a, Real.1 + Real.1, n)).rpow(Real.1 / (Real.1 + Real.1)) = Option.some(uu)
        and (abs_pow_sum(b, Real.1 + Real.1, n)).rpow(Real.1 / (Real.1 + Real.1)) = Option.some(vv)
        and partial(abs_product(a, b), n) <= uu * vv
    }
} by {
    Real.0 < Real.1
    lt_add_left(Real.1, Real.0, Real.1)
    Real.1 + Real.0 < Real.1 + Real.1
    Real.1 + Real.0 = Real.1
    Real.1 < Real.1 + Real.1
    Real.1 + Real.1 > Real.1
    Real.1 + Real.1 != Real.0
    div_add_same_denom(Real.1, Real.1, Real.1 + Real.1)
    Real.1 / (Real.1 + Real.1) + Real.1 / (Real.1 + Real.1) = (Real.1 + Real.1) / (Real.1 + Real.1)
    mul_div_cancel(Real.1, Real.1 + Real.1)
    (Real.1 + Real.1) * (Real.1 / (Real.1 + Real.1)) = Real.1
    (Real.1 + Real.1) / (Real.1 + Real.1) = Real.1
    Real.1 / (Real.1 + Real.1) + Real.1 / (Real.1 + Real.1) = Real.1
    finite_holder_exists(a, b, n, Real.1 + Real.1, Real.1 + Real.1)
    exists(uu: Real, vv: Real) {
        (abs_pow_sum(a, Real.1 + Real.1, n)).rpow(Real.1 / (Real.1 + Real.1)) = Option.some(uu)
        and (abs_pow_sum(b, Real.1 + Real.1, n)).rpow(Real.1 / (Real.1 + Real.1)) = Option.some(vv)
        and partial(abs_product(a, b), n) <= uu * vv
    }
}
