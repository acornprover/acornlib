from nat import Nat

numerals Nat

/// The index of the pair term at an odd position.
theorem cos_two_tail_pair_index_odd(m: Nat) {
    Nat.2 * m.suc + Nat.1 = Nat.2 * m + Nat.3
} by {
}

/// The index of the pair term at an even position.
theorem cos_two_tail_pair_index_even(m: Nat) {
    Nat.2 * m.suc + Nat.2 = Nat.2 * m + Nat.4
} by {
    from nat import mul_suc_right, add_comm, add_assoc
    mul_suc_right(Nat.2, m)
    Nat.2 * m.suc = Nat.2 + Nat.2 * m
    add_comm(Nat.2, Nat.2 * m)
    Nat.2 + Nat.2 * m = Nat.2 * m + Nat.2
    Nat.2 * m.suc = Nat.2 * m + Nat.2
    add_assoc(Nat.2 * m, Nat.2, Nat.2)
    Nat.2 * m + Nat.2 + Nat.2 = Nat.2 * m + (Nat.2 + Nat.2)
    Nat.2 + Nat.2 = Nat.4
    Nat.2 * m + (Nat.2 + Nat.2) = Nat.2 * m + Nat.4
    Nat.2 * m + Nat.2 + Nat.2 = Nat.2 * m + Nat.4
    Nat.2 * m.suc + Nat.2 = Nat.2 * m + Nat.4
}

/// The double of a successor and two more is four more than the double.
theorem cos_two_tail_double_suc_index(m: Nat) {
    Nat.2 * m.suc.suc = Nat.2 * m + Nat.4
} by {
}

/// A successor of the odd index shifts the tail index.
theorem cos_two_tail_odd_suc(m: Nat) {
    (Nat.2 * m + Nat.3).suc = Nat.2 * m + Nat.4
} by {
}

/// Adding two after the odd index shifts the tail index.
theorem cos_two_tail_odd_add_two(m: Nat) {
    Nat.2 * m + Nat.3 + Nat.2 = Nat.2 * m + Nat.5
} by {
    from nat import add_assoc
    add_assoc(Nat.2 * m, Nat.3, Nat.2)
    Nat.2 * m + Nat.3 + Nat.2 = Nat.2 * m + (Nat.3 + Nat.2)
    Nat.3 + Nat.2 = Nat.5
    Nat.2 * m + (Nat.3 + Nat.2) = Nat.2 * m + Nat.5
    Nat.2 * m + Nat.3 + Nat.2 = Nat.2 * m + Nat.5
}

/// The odd tail index is at least one.
theorem cos_two_tail_odd_ge_one(m: Nat) {
    Nat.1 <= Nat.2 * m + Nat.3
} by {
    from nat import lte_add_left, add_comm, lte_suc_suc, lt_suc, lt_trans, lte_trans
    Nat.0 <= Nat.2 * m
    lte_add_left(Nat.3, Nat.0, Nat.2 * m)
    Nat.3 + Nat.0 <= Nat.3 + Nat.2 * m
    Nat.3 + Nat.0 = Nat.3
    Nat.3 <= Nat.3 + Nat.2 * m
    add_comm(Nat.3, Nat.2 * m)
    Nat.3 + Nat.2 * m = Nat.2 * m + Nat.3
    Nat.3 <= Nat.2 * m + Nat.3
    Nat.0 <= Nat.1
    lte_suc_suc(Nat.0, Nat.1)
    Nat.1 <= Nat.2
    lte_suc_suc(Nat.1, Nat.2)
    Nat.2 <= Nat.3
    lte_trans(Nat.1, Nat.2, Nat.3)
    Nat.1 <= Nat.3
    lte_trans(Nat.1, Nat.3, Nat.2 * m + Nat.3)
    Nat.1 <= Nat.2 * m + Nat.3
}

/// The odd partial-sum index shifts by two at a successor.
theorem cos_two_tail_odd_partial_index(k: Nat) {
    Nat.2 * k.suc + Nat.3 = (Nat.2 * k + Nat.3).suc.suc
} by {
    from nat import mul_suc_right, add_comm, add_assoc
    mul_suc_right(Nat.2, k)
    Nat.2 * k.suc = Nat.2 + Nat.2 * k
    add_comm(Nat.2, Nat.2 * k)
    Nat.2 + Nat.2 * k = Nat.2 * k + Nat.2
    Nat.2 * k.suc = Nat.2 * k + Nat.2
    add_assoc(Nat.2 * k, Nat.2, Nat.3)
    Nat.2 * k + Nat.2 + Nat.3 = Nat.2 * k + (Nat.2 + Nat.3)
    Nat.2 + Nat.3 = Nat.5
    Nat.2 * k + (Nat.2 + Nat.3) = Nat.2 * k + Nat.5
    Nat.2 * k + Nat.2 + Nat.3 = Nat.2 * k + Nat.5
    Nat.2 * k.suc + Nat.3 = Nat.2 * k + Nat.5
    add_assoc(Nat.2 * k, Nat.3, Nat.2)
    Nat.2 * k + Nat.3 + Nat.2 = Nat.2 * k + (Nat.3 + Nat.2)
    Nat.3 + Nat.2 = Nat.5
    Nat.2 * k + Nat.3 + Nat.2 = Nat.2 * k + Nat.5
    (Nat.2 * k + Nat.3).suc.suc = Nat.2 * k + Nat.3 + Nat.2
    (Nat.2 * k + Nat.3).suc.suc = Nat.2 * k + Nat.5
    Nat.2 * k.suc + Nat.3 = (Nat.2 * k + Nat.3).suc.suc
}

/// The tail index at two more is two more.
theorem cos_two_tail_index_add_two(m: Nat) {
    Nat.2 * m + Nat.2 + Nat.2 = Nat.2 * m + Nat.4
} by {
    from nat import add_assoc
    add_assoc(Nat.2 * m, Nat.2, Nat.2)
    Nat.2 * m + Nat.2 + Nat.2 = Nat.2 * m + (Nat.2 + Nat.2)
    Nat.2 + Nat.2 = Nat.4
    Nat.2 * m + (Nat.2 + Nat.2) = Nat.2 * m + Nat.4
    Nat.2 * m + Nat.2 + Nat.2 = Nat.2 * m + Nat.4
}

/// The tail index at one more is one more.
theorem cos_two_tail_index_add_one(m: Nat) {
    Nat.2 * m + Nat.1 + Nat.2 = Nat.2 * m + Nat.3
} by {
    from nat import add_assoc
    add_assoc(Nat.2 * m, Nat.1, Nat.2)
    Nat.2 * m + Nat.1 + Nat.2 = Nat.2 * m + (Nat.1 + Nat.2)
    Nat.1 + Nat.2 = Nat.3
    Nat.2 * m + (Nat.1 + Nat.2) = Nat.2 * m + Nat.3
    Nat.2 * m + Nat.1 + Nat.2 = Nat.2 * m + Nat.3
}


/// Two times eight is sixteen.
theorem nat_mul_2_8 {
    Nat.2 * Nat.8 = Nat.16
} by {
}

/// Three times eight is twenty-four.
theorem nat_mul_3_8 {
    Nat.3 * Nat.8 = Nat.24
} by {
}
