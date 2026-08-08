from algebra.add import Add
from lte import LTE
from nat import Nat
from algebra.neg import Neg
from int import Int, abs
from rat import Rat

// This file contains the initial definition of real numbers, based on Dedekind cuts.
// It handles addition, negation, subtraction, and ordering.

/// True if a function on rationals partitions them into two non-empty sets.
define is_cut(f: Rat -> Bool) -> Bool {
    exists(x: Rat) {
        f(x)
    } and exists(x: Rat) {
        not f(x)
    }
}

/// True if a set is downward-closed (contains all smaller elements).
define is_lower(f: Rat -> Bool) -> Bool {
    forall(x: Rat, y: Rat) {
        f(y) and x < y implies f(x)
    }
}

theorem lower_contains_smaller(f: Rat -> Bool, x: Rat, y: Rat) {
    is_lower(f) and f(y) and x < y implies f(x)
} by {
    is_lower(f) = forall(u: Rat, v: Rat) {
        f(v) and u < v implies f(u)
    }
    if f(y) and x < y {
        forall(u: Rat, v: Rat) {
            f(v) and u < v implies f(u)
        }
        f(y) and x < y implies f(x)
        f(x)
    }
}

/// True if x is the greatest element in the set defined by f.
define is_greatest(f: Rat -> Bool, x: Rat) -> Bool {
    f(x) and forall(y: Rat) {
        f(y) implies y <= x
    }
}

/// True if the set defined by f has a greatest element.
define has_greatest(f: Rat -> Bool) -> Bool {
    exists(x: Rat) {
        is_greatest(f, x)
    }
}

/// True if a function represents a valid Dedekind cut defining a real number.
define is_dedekind_cut(f: Rat -> Bool) -> Bool {
    is_cut(f) and is_lower(f) and not has_greatest(f)
}

// "All numbers y such that x is greater then y" is the cut that embeds x.

theorem gt_is_cut(r: Rat) {
    is_cut(r.gt)
} by {
    not r > r
}

theorem gt_is_lower(r: Rat) {
    is_lower(r.gt)
} by {
    forall(x: Rat, y: Rat) {
        if r.gt(y) and x < y {
            r > y
            r > x
        }
    }
}

theorem gt_has_no_greatest(r: Rat) {
    not has_greatest(r.gt)
} by {
    if has_greatest(r.gt) {
        let q: Rat satisfy {
            is_greatest(r.gt, q)
        }
        r.gt(q)
        r > q
        let diff = r - q
        diff.is_positive

        // z will be a counterexample.
        // It's constructed to be larger than q.
        let z = q + diff / Rat.2
        q < z

        // z is still less than r, though.
        diff / Rat.2 + diff / Rat.2 = diff
        z + diff / Rat.2 = q + diff / Rat.2 + diff / Rat.2
        q + diff / Rat.2 + diff / Rat.2 = q + diff
        q + diff = q + (r - q)
        q + (r - q) = r
        z + diff / Rat.2 = r
        r > z

        // Since q is the greatest satisfying r.gt(_). But this contradicts.
        false
    }
}

theorem gt_is_dedekind_cut(r: Rat) {
    is_dedekind_cut(r.gt)
} by {
}

/// Real numbers are defined by a Dedekind cut. Specifically, using the `gt_rat` function which
/// specifies which rationals they are greater than.
structure Real {
    /// True if this real number is greater than the given rational number.
    gt_rat: Rat -> Bool
} constraint {
    is_dedekind_cut(gt_rat)
}

let real_from_rat(r: Rat) -> a: Real satisfy {
    Real.new(r.gt) = Option.some(a)
}

/// Real extensionality: two real numbers are equal when they have the same cut.
theorem real_ext(a: Real, b: Real) {
    (forall(r: Rat) { a.gt_rat(r) = b.gt_rat(r) }) implies a = b
} by {
    if forall(r: Rat) { a.gt_rat(r) = b.gt_rat(r) } {
        a.gt_rat = b.gt_rat
    }
}

attributes Real {
    /// Converts a rational number to a real number.
    let from_rat = real_from_rat

    /// Real extensionality from pointwise equality of the cut.
    let ext = real_ext
}

from algebra.zero import Zero

instance Real: Zero {
    let 0: Real = Real.from_rat(Rat.0)
}

attributes Real {
    /// True if this real number is positive (greater than zero).
    define is_positive(self) -> Bool {
        self.gt_rat(Rat.0)
    }

    /// True if this real number is negative (less than zero).
    define is_negative(self) -> Bool {
        self != Real.0 and not self.is_positive
    }
}

/// The less-than-or-equal-to relation for real numbers.
instance Real: LTE {
    define lte(self, other: Real) -> Bool {
        forall(r: Rat) {
            self.gt_rat(r) implies other.gt_rat(r)
        }
    }
}

theorem lte_self(r: Real) {
    r <= r
}

theorem lte_trans(a: Real, b: Real, c: Real) {
    a <= b and b <= c implies a <= c
}

/// Transitivity with equality: if a <= b and b = c, then a <= c.
theorem lte_trans_eq(a: Real, b: Real, c: Real) {
    a <= b and b = c implies a <= c
}

theorem gt_rat_sorts(z: Real, r1: Rat, r2: Rat) {
    z.gt_rat(r1) and not z.gt_rat(r2) implies r1 <= r2
} by {
    if r2 < r1 {
        is_lower(z.gt_rat)
        z.gt_rat(r2)
        false
    }
}

theorem lte_or_gte(a: Real, b: Real) {
    a <= b or b <= a
} by {
    if not a <= b {
        let r1: Rat satisfy {
            a.gt_rat(r1) and not b.gt_rat(r1)
        }
        forall(r2: Rat) {
            if b.gt_rat(r2) {
                r2 <= r1
                r2 < r1 or r2 = r1
                is_lower(a.gt_rat)
                a.gt_rat(r2)
            }
        }
    }
}

theorem lte_both_ways_imp_eq(a: Real, b: Real) {
    a <= b and b <= a implies a = b
} by {
    forall(r: Rat) {
        if a.gt_rat(r) {
            b.gt_rat(r)
            a.gt_rat(r) = b.gt_rat(r)
        } else {
            if b.gt_rat(r) {
                false
            }
            a.gt_rat(r) = b.gt_rat(r)
        }
    }
    Real.ext(a, b)
}

// The real numbers form a total order.

from data.basic.relation_basic import is_reflexive, is_transitive, is_antisymmetric

theorem real_is_reflexive {
    is_reflexive(Real.lte)
} by {
    forall(t: Real) {
    }
}

theorem real_is_transitive {
    is_transitive(Real.lte)
} by {
    if not is_transitive(Real.lte) {
        let (a: Real, b: Real, c: Real) satisfy {
            a <= b and b <= c and not (a <= c)
        }
        false
    }
}

theorem real_is_antisymmetric {
    is_antisymmetric(Real.lte)
}

from order import PartialOrder, LinearOrder, min_eq_left_of_lt, min_eq_right_of_not_lt, max_eq_left_of_gt, max_eq_right_of_not_gt, lt_min_iff, max_lt_iff, lte_min_of_bounds, lte_max_left, lte_max_right, max_lte_of_upper_bounds, min_max_distrib_left, max_min_distrib_left
from lattice import Meet, Join, MeetSemilattice, JoinSemilattice, Lattice, DistribLattice

instance Real: PartialOrder

instance Real: LinearOrder

theorem gt_imp_from_rat_gt(r1: Rat, r2: Rat) {
    r1 > r2 implies Real.from_rat(r1).gt_rat(r2)
}

theorem from_rat_gt_imp_gt(r1: Rat, r2: Rat) {
    Real.from_rat(r1).gt_rat(r2) implies r1 > r2
}

theorem not_gt_rat_self(r: Rat) {
    not Real.from_rat(r).gt_rat(r)
} by {
    if Real.from_rat(r).gt_rat(r) {
        r > r
    }
}

theorem zero_not_positive {
    not Real.0.is_positive
}

theorem gte_self(r: Real) {
    r >= r
}

theorem gt_rat_imp_gt_from_rat(a: Real, r: Rat) {
    a.gt_rat(r) implies a > Real.from_rat(r)
}

theorem gt_from_rat_imp_gt_rat(a: Real, r: Rat) {
    a > Real.from_rat(r) implies a.gt_rat(r)
} by {
    not a <= Real.from_rat(r)
    let r2: Rat satisfy {
        a.gt_rat(r2) and not Real.from_rat(r).gt_rat(r2)
    }
    not r > r2
    r <= r2
    is_lower(a.gt_rat)
    a.gt_rat(r)
}

theorem pos_gt_zero(a: Real) {
    a.is_positive implies a > Real.0
}

theorem gt_zero_imp_pos(a: Real) {
    a > Real.0 implies a.is_positive
}

theorem neg_lt_zero(a: Real) {
    a.is_negative implies a < Real.0
}

theorem lt_zero_imp_neg(a: Real) {
    a < Real.0 implies a.is_negative
}

// Informally, z1 + z2 > r.
define add_gt(z1: Real, z2: Real, r: Rat) -> Bool {
    exists(r1: Rat, r2: Rat) {
        z1.gt_rat(r1) and z2.gt_rat(r2) and r = r1 + r2
    }
}

theorem add_gt_symm(z1: Real, z2: Real, r: Rat) {
    add_gt(z1, z2, r) implies add_gt(z2, z1, r)
} by {
    let (r1: Rat, r2: Rat) satisfy {
        z1.gt_rat(r1) and z2.gt_rat(r2) and r = r1 + r2
    }
    r = r2 + r1
    add_gt(z2, z1, r)
}

theorem exists_lesser_rat(z: Real) {
    exists(r: Rat) {
        z.gt_rat(r)
    }
} by {
    is_cut(z.gt_rat)
}

theorem exists_gte_rat(z: Real) {
    exists(r: Rat) {
        not z.gt_rat(r)
    }
} by {
    is_cut(z.gt_rat)
}

theorem add_gt_is_cut(z1: Real, z2: Real) {
    is_cut(add_gt(z1, z2))
} by {
    // Prove there's something lower
    let lower1: Rat satisfy {
        z1.gt_rat(lower1)
    }
    let lower2: Rat satisfy {
        z2.gt_rat(lower2)
    }
    add_gt(z1, z2, lower1 + lower2)

    // Prove there's something higher
    let upper1: Rat satisfy {
        not z1.gt_rat(upper1)
    }
    let upper2: Rat satisfy {
        not z2.gt_rat(upper2)
    }
    if add_gt(z1, z2, upper1 + upper2) {
        let (f1: Rat, f2: Rat) satisfy {
            z1.gt_rat(f1) and z2.gt_rat(f2) and upper1 + upper2 = f1 + f2
        }
        f1 <= upper1
        f2 <= upper2
        if f1 < upper1 {
            f1 + f2 < upper1 + f2
            upper1 + f2 <= upper1 + upper2
            f1 + f2 < upper1 + upper2
            false
        } else {
            f1 <= upper1 and not (f1 < upper1)
            f1 = upper1
            false
        }
    }
}

theorem add_gt_is_lower(z1: Real, z2: Real) {
    is_lower(add_gt(z1, z2))
} by {
    forall(r1: Rat, r2: Rat) {
        if add_gt(z1, z2, r2) and r1 < r2 {
            let (f1: Rat, f2: Rat) satisfy {
                z1.gt_rat(f1) and z2.gt_rat(f2) and r2 = f1 + f2
            }
            f2 + -(r2 - r1) < f2
            z2.gt_rat(f2 + -(r2 - r1))
            add_gt(z1, z2, r1)
        }
    }
}

theorem add_gt_has_no_greatest(z1: Real, z2: Real) {
    not has_greatest(add_gt(z1, z2))
} by {
    if has_greatest(add_gt(z1, z2)) {
        let q: Rat satisfy {
            is_greatest(add_gt(z1, z2), q)
        }
        add_gt(z1, z2, q)
        let (f1: Rat, f2: Rat) satisfy {
            z1.gt_rat(f1) and z2.gt_rat(f2) and q = f1 + f2
        }
        not has_greatest(z1.gt_rat)
        not is_greatest(z1.gt_rat, f1)
        let g1: Rat satisfy {
            z1.gt_rat(g1) and not g1 <= f1
        }
        g1 > f1
        g1 + f2 > f1 + f2
        q < g1 + f2
        add_gt(z1, z2, g1 + f2)
        false
    }
}

theorem add_gt_is_dedekind_cut(z1: Real, z2: Real) {
    is_dedekind_cut(add_gt(z1, z2))
}

let add(a: Real, b: Real) -> c: Real satisfy {
    Real.new(add_gt(a, b)) = Option.some(c)
}

/// The sum of two real numbers.
instance Real: Add {
    let add = add
}

theorem add_gt_rat(z1: Real, z2: Real, r1: Rat, r2: Rat) {
    z1.gt_rat(r1) and z2.gt_rat(r2) implies (z1 + z2).gt_rat(r1 + r2)
}

theorem add_comm(a: Real, b: Real) {
    a + b = b + a
} by {
    forall(r: Rat) {
        if add_gt(a, b, r) {
            add_gt(a, b, r) = add_gt(b, a, r)
        } else {
            if add_gt(b, a, r) {
                false
            }
            add_gt(a, b, r) = add_gt(b, a, r)
        }
    }
    add_gt(a, b) = add_gt(b, a)
}

theorem gt_rat_adding_three(z1: Real, z2: Real, z3: Real, q: Rat) {
    (z1 + z2 + z3).gt_rat(q) implies exists(r1: Rat, r2: Rat, r3: Rat) {
        z1.gt_rat(r1) and z2.gt_rat(r2) and z3.gt_rat(r3) and q = r1 + r2 + r3
    }
} by {
    add_gt(z1 + z2, z3, q)
    let (r12: Rat, r3: Rat) satisfy {
        (z1 + z2).gt_rat(r12) and z3.gt_rat(r3) and q = r12 + r3
    }
    add_gt(z1, z2, r12)
    let (r1: Rat, r2: Rat) satisfy {
        z1.gt_rat(r1) and z2.gt_rat(r2) and r12 = r1 + r2
    }
    q = r12 + r3
    r12 = r1 + r2
    q = (r1 + r2) + r3
    q = r1 + r2 + r3
    z1.gt_rat(r1) and z2.gt_rat(r2) and z3.gt_rat(r3) and q = r1 + r2 + r3
    exists(a1: Rat, a2: Rat, a3: Rat) {
        z1.gt_rat(a1) and z2.gt_rat(a2) and z3.gt_rat(a3) and q = a1 + a2 + a3
    }
}

theorem gt_rat_adding_three_converse(z1: Real, z2: Real, z3: Real, q: Rat) {
    exists(r1: Rat, r2: Rat, r3: Rat) {
        z1.gt_rat(r1) and z2.gt_rat(r2) and z3.gt_rat(r3) and q = r1 + r2 + r3
    } implies (z1 + z2 + z3).gt_rat(q)
} by {
    let (r1: Rat, r2: Rat, r3: Rat) satisfy {
        z1.gt_rat(r1) and z2.gt_rat(r2) and z3.gt_rat(r3) and q = r1 + r2 + r3
    }
    q = (r1 + r2) + r3
    exists(s1: Rat, s2: Rat) {
        z1.gt_rat(s1) and z2.gt_rat(s2) and r1 + r2 = s1 + s2
    }
    add_gt(z1, z2, r1 + r2)
    (z1 + z2).gt_rat(r1 + r2)
    exists(t1: Rat, t2: Rat) {
        (z1 + z2).gt_rat(t1) and z3.gt_rat(t2) and q = t1 + t2
    }
    add_gt(z1 + z2, z3, q)
}

theorem add_assoc(a: Real, b: Real, c: Real) {
    a + b + c = a + (b + c)
} by {
    forall(r: Rat) {
        if (a + b + c).gt_rat(r) {
            let (ra: Rat, rb: Rat, rc: Rat) satisfy {
                a.gt_rat(ra) and b.gt_rat(rb) and c.gt_rat(rc) and r = ra + rb + rc
            }
            (a + b + c).gt_rat(r) = (a + (b + c)).gt_rat(r)
        } else {
            if (a + (b + c)).gt_rat(r) {
                add_gt(a, b + c, r)
                let (ra: Rat, rbc: Rat) satisfy {
                    a.gt_rat(ra) and (b + c).gt_rat(rbc) and r = ra + rbc
                }
                add_gt(b, c, rbc)
                let (rb: Rat, rc: Rat) satisfy {
                    b.gt_rat(rb) and c.gt_rat(rc) and rbc = rb + rc
                }
                r = ra + rb + rc
                (a + b + c).gt_rat(r)
                false
            }
            (a + b + c).gt_rat(r) = (a + (b + c)).gt_rat(r)
        }
    }
    Real.ext(a + b + c, a + (b + c))
}

theorem gt_imp_not_lte(a: Real, b: Real) {
    a > b implies not a <= b
}

theorem not_lte_imp_gt(a: Real, b: Real) {
    not a <= b implies a > b
}

theorem gte_imp_not_lt(a: Real, b: Real) {
    a >= b implies not a < b
}

theorem not_lt_imp_gte(a: Real, b: Real) {
    not a < b implies a >= b
}

theorem rat_separating(a: Real, b: Real) {
    a < b implies exists(r: Rat) {
        b.gt_rat(r) and not a.gt_rat(r)
    }
}

theorem rat_between_rat_and_real(z: Real, r1: Rat) {
    z.gt_rat(r1) implies exists(r2: Rat) {
        z.gt_rat(r2) and r1 < r2
    }
} by {
    not is_greatest(z.gt_rat, r1)
    exists(r2: Rat) {
        z.gt_rat(r2) and not r2 <= r1
    }
}

// Strict version of rat_separating.
theorem rat_between_reals(a: Real, b: Real) {
    a < b implies exists(r: Rat) {
        a < Real.from_rat(r) and Real.from_rat(r) < b
    }
} by {
    // r1 is less than b, but it might be equal to a.
    // We need to use the "no greatest" property of b.gt_rat.
    let r1: Rat satisfy {
        b.gt_rat(r1) and not a.gt_rat(r1)
    }
    let r2: Rat satisfy {
        b.gt_rat(r2) and r1 < r2
    }
    a < Real.from_rat(r2)
}

theorem rat_between_reals_gt(a: Real, b: Real) {
    a > b implies
    exists(r: Rat) {
        a > Real.from_rat(r) and Real.from_rat(r) > b
    }
} by {
}

theorem add_gt_trans(z1: Real, z2: Real, r1: Rat, r2: Rat) {
    add_gt(z1, z2, r1) and r1 > r2 implies add_gt(z1, z2, r2)
}

theorem add_gt_imp_gt_from_rat(z1: Real, z2: Real, r: Rat) {
    add_gt(z1, z2, r) implies z1 + z2 > Real.from_rat(r)
}

theorem gt_from_rat_imp_add_gt(z1: Real, z2: Real, r: Rat) {
    z1 + z2 > Real.from_rat(r) implies add_gt(z1, z2, r)
}

theorem lt_lte_trans(a: Real, b: Real, c: Real) {
    a < b and b <= c implies a < c
}

theorem lte_lt_trans(a: Real, b: Real, c: Real) {
    a <= b and b < c implies a < c
}

theorem lt_trans(a: Real, b: Real, c: Real) {
    a < b and b < c implies a < c
}

theorem add_gt_from_rat_imp_rat_add_gt(p: Rat, q: Rat, r: Rat) {
    add_gt(Real.from_rat(p), Real.from_rat(q), r) implies p + q > r
} by {
    let (rp: Rat, rq: Rat) satisfy {
        Real.from_rat(p).gt_rat(rp) and Real.from_rat(q).gt_rat(rq) and r = rp + rq
    }
}

theorem rat_add_gt_imp_add_gt_from_rat(p: Rat, q: Rat, r: Rat) {
    p + q > r implies add_gt(Real.from_rat(p), Real.from_rat(q), r)
} by {
    let d = p + q - r
    d.is_positive
    let rp = p - d / Rat.2
    (d / Rat.2).is_positive
    p > rp
    Real.from_rat(p).gt_rat(rp)
    let rq = q - d / Rat.2
    q > rq
    Real.from_rat(q).gt_rat(rq)
    let h = d / Rat.2
    h + h = d
    rp = p - h
    rq = q - h
    rp + rq = (p - h) + (q - h)
    (p - h) + (q - h) = p - h + q - h
    p - h + q - h = p + q - h - h
    p + q - h - h = p + q - (h + h)
    p + q - (h + h) = p + q - d
    rp + rq = p + q - d
    d = p + q - r
    p + q - d = p + q - (p + q - r)
    p + q - (p + q - r) = p + q - p - q + r
    p + q - p - q + r = r
    p + q - d = r
    rp + rq = r
    add_gt(Real.from_rat(p), Real.from_rat(q), r)
}

theorem add_from_rat(p: Rat, q: Rat) {
    Real.from_rat(p) + Real.from_rat(q) = Real.from_rat(p + q)
} by {
    if Real.from_rat(p) + Real.from_rat(q) < Real.from_rat(p + q) {
        let r: Rat satisfy {
            Real.from_rat(p) + Real.from_rat(q) < Real.from_rat(r) and Real.from_rat(r) < Real.from_rat(p + q)
        }
        r < p + q
        p + q > r
        add_gt(Real.from_rat(p), Real.from_rat(q), r)
        (Real.from_rat(p) + Real.from_rat(q)).gt_rat(r)
        Real.from_rat(p) + Real.from_rat(q) > Real.from_rat(r)
        false
    }
    if Real.from_rat(p + q) < Real.from_rat(p) + Real.from_rat(q) {
        let r: Rat satisfy {
            Real.from_rat(p + q) < Real.from_rat(r) and Real.from_rat(r) < Real.from_rat(p) + Real.from_rat(q)
        }
        r > p + q
        add_gt(Real.from_rat(p), Real.from_rat(q), r)
        p + q > r
        false
    }
}

theorem lte_add_right(a: Real, b: Real, c: Real) {
    a <= b implies a + c <= b + c
} by {
    forall(r: Rat) {
        if (a + c).gt_rat(r) {
            add_gt(a, c, r)
            let (ra: Rat, rc: Rat) satisfy {
                a.gt_rat(ra) and c.gt_rat(rc) and r = ra + rc
            }
            (b + c).gt_rat(r)
        }
    }
}

theorem lte_add_left(a: Real, b: Real, c: Real) {
    a <= b implies c + a <= c + b
}

/// Adding two inequalities preserves the inequality.
theorem add_lte_add(a: Real, b: Real, c: Real, d: Real) {
    a <= b and c <= d implies a + c <= b + d
}

theorem lt_add_converse(a: Real, b: Real, c: Real) {
    a + c < b + c implies a < b
}

theorem add_from_rat_zero(r: Rat) {
    Real.from_rat(r) + Real.0 = Real.from_rat(r)
}

theorem add_zero_right(a: Real) {
    a + Real.0 = a
} by {
    if a + Real.0 < a {
        let r: Rat satisfy {
            a + Real.0 < Real.from_rat(r) and Real.from_rat(r) < a
        }
        Real.from_rat(r) + Real.0 = Real.from_rat(r)
        a + Real.0 < Real.from_rat(r) + Real.0
        a < Real.from_rat(r)
        false
    }
    if a < a + Real.0 {
        // Real.0 is not positive, so a + Real.0 cannot be greater than a
        not Real.0.is_positive
        // By lte_or_gte, either a + Real.0 <= a or a <= a + Real.0
        // We're in the case a < a + Real.0, so a + Real.0 > a
        // But a + Real.0 > a would require Real.0 to be positive
        // Let's show this leads to contradiction by finding r between them
        let r: Rat satisfy {
            a < Real.from_rat(r) and Real.from_rat(r) < a + Real.0
        }
        // We have a < Real.from_rat(r) and Real.from_rat(r) < a + Real.0
        // Real.from_rat(r) + Real.0 = Real.from_rat(r) by add_from_rat_zero
        // From a < Real.from_rat(r), and Real.0 is not positive
        // We should get a + Real.0 <= Real.from_rat(r) + Real.0 = Real.from_rat(r)
        // But this contradicts Real.from_rat(r) < a + Real.0
        a <= Real.from_rat(r)
        a + Real.0 <= Real.from_rat(r) + Real.0
        Real.from_rat(r) + Real.0 = Real.from_rat(r)
        a + Real.0 <= Real.from_rat(r)
        false
    }
}

theorem add_zero_left(a: Real) {
    Real.0 + a = a
}

theorem from_rat_maintains_lt(p: Rat, q: Rat) {
    p < q implies Real.from_rat(p) < Real.from_rat(q)
}

theorem add_lt_lt(a: Real, b: Real, c: Real, d: Real) {
    a < b and c < d implies a + c < b + d
} by {
    // Find two rationals between each pair of real numbers
    let rab1: Rat satisfy {
        a < Real.from_rat(rab1) and Real.from_rat(rab1) < b
    }
    let rab2: Rat satisfy {
        Real.from_rat(rab1) < Real.from_rat(rab2) and Real.from_rat(rab2) < b
    }
    let rcd1: Rat satisfy {
        c < Real.from_rat(rcd1) and Real.from_rat(rcd1) < d
    }
    let rcd2: Rat satisfy {
        Real.from_rat(rcd1) < Real.from_rat(rcd2) and Real.from_rat(rcd2) < d
    }

    // We can get a non-strict inequality in reals. Here's the left side.
    Real.from_rat(rab1) + c <= Real.from_rat(rab1) + Real.from_rat(rcd1)
    a + c <= Real.from_rat(rab1) + Real.from_rat(rcd1)

    // Now we get the right side.
    Real.from_rat(rab2) + Real.from_rat(rcd2) <= Real.from_rat(rab2) + d

    // Then we can get the strict inequality in the middle from the all-rational formula.
    rab1 < rab2
    rcd1 < rcd2
    rab1 + rcd1 < rab1 + rcd2
    rab1 + rcd2 < rab2 + rcd2
    rab1 + rcd1 < rab2 + rcd2
    Real.from_rat(rab1) + Real.from_rat(rcd1) = Real.from_rat(rab1 + rcd1)
    Real.from_rat(rab2) + Real.from_rat(rcd2) = Real.from_rat(rab2 + rcd2)
    Real.from_rat(rab1 + rcd1) < Real.from_rat(rab2 + rcd2)
    Real.from_rat(rab1) + Real.from_rat(rcd1) < Real.from_rat(rab2) + Real.from_rat(rcd2)
    a + c < Real.from_rat(rab2) + Real.from_rat(rcd2)
    Real.from_rat(rab2) + Real.from_rat(rcd2) <= b + d
}

theorem lte_some_rat(a: Real) {
    exists(r: Rat) {
        a <= Real.from_rat(r)
    }
}

theorem lt_some_rat(a: Real) {
    exists(r: Rat) {
        a < Real.from_rat(r)
    }
}

theorem gt_some_rat(a: Real) {
    exists(r: Rat) {
        a > Real.from_rat(r)
    }
}

attributes Real {
    /// Converts an integer to a real number.
    let from_int = function(n: Int) {
        Real.from_rat(Rat.from_int(n))
    }
}

theorem from_rat_maintains_lte(p: Rat, q: Rat) {
    p <= q implies Real.from_rat(p) <= Real.from_rat(q)
}

theorem gt_some_int(a: Real) {
    exists(n: Int) {
        a > Real.from_int(n)
    }
} by {
    let r: Rat satisfy {
        a > Real.from_rat(r)
    }
    Real.from_int(r.floor) <= Real.from_rat(r)
}

theorem lt_some_int(a: Real) {
    exists(n: Int) {
        a < Real.from_int(n)
    }
} by {
    let r: Rat satisfy {
        a < Real.from_rat(r)
    }
    let n: Int satisfy {
        r < Rat.from_int(n)
    }
}

theorem real_neg_imp_rat_neg(r: Rat) {
    Real.from_rat(r).is_negative implies r.is_negative
} by {
    r < Rat.0
}

theorem lt_some_int_cancel(m: Int, n: Int) {
    Real.from_int(m) < Real.from_int(n) implies m < n
} by {
    Rat.from_int(m) < Rat.from_int(n)
}

theorem floor_exists(a: Real) {
    exists(n: Int) {
        Real.from_int(n) <= a and a < Real.from_int(n + Int.1)
    }
} by {
    if not floor_exists(a) {
        // We will show by induction that a is greater than all integers.
        // First let's find a base case.
        let m: Int satisfy {
            a > Real.from_int(m)
        }

        // Now we define the function to induct on.
        let f = function(k: Nat) {
            Real.from_int(m + Int.from_nat(k)) <= a
        }
        f(Nat.0)

        // Inductive step
        forall(x: Nat) {
            if f(x) {
                Real.from_int(m + Int.from_nat(x) + Int.1) <= a
                Int.from_nat(x.suc) = Int.from_nat(x) + Int.1
                m + Int.from_nat(x.suc) = m + Int.from_nat(x) + Int.1
                Real.from_int(m + Int.from_nat(x.suc)) <= a
                f(x.suc)
            }
        }

        // By induction we have f for all k

        // But m plus some really big number has to be greater than a.
        let big: Int satisfy {
            a < Real.from_int(big)
        }
        let k = abs(m) + abs(big)
        f(k)
        Real.from_int(m + Int.from_nat(k)) <= a
        -m <= Int.from_nat(abs(m))
        Int.0 <= m + Int.from_nat(abs(m))
        Int.from_nat(abs(big)) <= m + Int.from_nat(abs(m)) + Int.from_nat(abs(big))
        big <= Int.from_nat(abs(big))
        big <= m + Int.from_nat(abs(m)) + Int.from_nat(abs(big))
        Int.from_nat(abs(m)) + Int.from_nat(abs(big)) = Int.from_nat(k)
        big <= m + Int.from_nat(k)
        Real.from_int(big) <= Real.from_int(m + Int.from_nat(k))
        Real.from_int(big) <= a
        false
    }
}

let floor(a: Real) -> n: Int satisfy {
    Real.from_int(n) <= a and a < Real.from_int(n + Int.1)
}

from algebra.one import One

instance Real: One {
    let 1: Real = Real.from_rat(Rat.1)
}

attributes Real {
    /// One half (1/2).
    let one_half: Real = Real.from_rat(Rat.2.inverse)
}

theorem one_half_positive {
    Real.0 < Real.one_half
} by {
    Rat.2.inverse.is_positive
    Rat.0 < Rat.2.inverse
}

theorem one_half_plus_one_half {
    Real.one_half + Real.one_half = Real.1
} by {
    Real.one_half = Real.from_rat(Rat.2.inverse)
    Real.one_half + Real.one_half = Real.from_rat(Rat.2.inverse) + Real.from_rat(Rat.2.inverse)
    Real.from_rat(Rat.2.inverse) + Real.from_rat(Rat.2.inverse) = Real.from_rat(Rat.2.inverse + Rat.2.inverse)
    Rat.2.inverse + Rat.2.inverse = Rat.1
}

theorem add_from_int(m: Int, n: Int) {
    Real.from_int(m) + Real.from_int(n) = Real.from_int(m + n)
}

theorem lt_add_one(a: Real) {
    a < a + Real.1
} by {
    let n = floor(a)
    Real.from_int(n) <= a
    a < Real.from_int(n + Int.1)
    if a = Real.from_int(n) {
        n < n + Int.1
        Real.from_int(n) < Real.from_int(n + Int.1)
        Real.from_int(n) + Real.1 = Real.from_int(n + Int.1)
        a < a + Real.1
    } else {
        // a > Real.from_int(n), so a >= Real.from_int(n)
        Real.from_int(n) <= a
        // By lte_add_right: Real.from_int(n) + Real.1 <= a + Real.1
        Real.from_int(n) + Real.1 <= a + Real.1
        // We also have a < Real.from_int(n + 1)
        Real.from_int(n + Int.1) = Real.from_int(n) + Real.1
        a < Real.from_int(n) + Real.1
        a < a + Real.1
    }
}

theorem lt_add_pos_int(a: Real, n: Int) {
    n.is_positive implies a < a + Real.from_int(n)
} by {
    Int.1 <= n
    Real.from_int(Int.1) <= Real.from_int(n)
    Real.1 <= Real.from_int(n)
    a + Real.1 <= a + Real.from_int(n)
    a < a + Real.1
    a < a + Real.from_int(n)
}

theorem lt_add_pos_rat(a: Real, r: Rat) {
    r.is_positive implies a < a + Real.from_rat(r)
} by {
    Real.0 <= Real.from_rat(r)
    if a = a + Real.from_rat(r) {
        // We can keep sticking on r's. Induct on f:
        let f = function(k: Nat) {
            a = a + Real.from_rat(r * Rat.from_int(Int.from_nat(k)))
        }

        // Base case
        f(Nat.0)

        // Inductive step
        forall(x: Nat) {
            if f(x) {
                a = a + Real.from_rat(r * Rat.from_int(Int.from_nat(x)))
                Real.from_rat(r) + Real.from_rat(r * Rat.from_int(Int.from_nat(x))) =
                    Real.from_rat(r + r * Rat.from_int(Int.from_nat(x)))
                r + r * Rat.from_int(Int.from_nat(x)) = r * (Rat.1 + Rat.from_int(Int.from_nat(x)))
                Rat.1 + Rat.from_int(Int.from_nat(x)) = Rat.from_int(Int.1 + Int.from_nat(x))
                Int.1 + Int.from_nat(x) = Int.from_nat(x.suc)
                r * (Rat.1 + Rat.from_int(Int.from_nat(x))) = r * Rat.from_int(Int.from_nat(x.suc))
                a + Real.from_rat(r) = a + Real.from_rat(r * Rat.from_int(Int.from_nat(x))) + Real.from_rat(r)
                a + Real.from_rat(r * Rat.from_int(Int.from_nat(x))) + Real.from_rat(r) =
                    a + Real.from_rat(r * Rat.from_int(Int.from_nat(x)) + r)
                a = a + Real.from_rat(r * Rat.from_int(Int.from_nat(x.suc)))
                f(x.suc)
            }
        }

        // Via induction
        forall(n: Nat) { f(n) }

        // Just add enough r's to get to an integer
        f(abs(r.denom))
        a = a + Real.from_rat(r * Rat.from_int(Int.from_nat(abs(r.denom))))
        r.denom > Int.0
        not r.denom.is_negative
        Int.from_nat(abs(r.denom)) = r.denom
        a = a + Real.from_rat(r * Rat.from_int(r.denom))
        false
    }
}

theorem lt_add_pos(a: Real, b: Real) {
    b.is_positive implies a < a + b
} by {
    let r: Rat satisfy {
        Real.0 < Real.from_rat(r) and Real.from_rat(r) < b
    }
    Real.from_rat(Rat.0) < Real.from_rat(r)
    Rat.0 < r
    r.is_positive
    a < a + Real.from_rat(r)
}

theorem lt_add_rat_right(a: Real, b: Real, r: Rat) {
    a < b implies a + Real.from_rat(r) < b + Real.from_rat(r)
} by {
  a + Real.from_rat(r) + Real.from_rat(-r) < b + Real.from_rat(r) + Real.from_rat(-r)
}

theorem lt_add_rat_left(a: Real, b: Real, r: Rat) {
    a < b implies Real.from_rat(r) + a < Real.from_rat(r) + b
}

theorem add_rat_eps_between(a: Real, b: Real) {
    a < b implies exists(eps: Rat) {
        eps.is_positive and a + Real.from_rat(eps) < b
    }
} by {
    let r1: Rat satisfy {
        a < Real.from_rat(r1) and Real.from_rat(r1) < b
    }
    let r2: Rat satisfy {
        Real.from_rat(r1) < Real.from_rat(r2) and Real.from_rat(r2) < b
    }
    let eps = r2 - r1
    r1 < r2

    // Combine them
    a + Real.from_rat(eps) < Real.from_rat(r1) + Real.from_rat(eps)
    Real.from_rat(r1) + Real.from_rat(eps) = Real.from_rat(r1 + eps)
    r1 + eps = r2
    Real.from_rat(r1) + Real.from_rat(eps) = Real.from_rat(r2)
    a + Real.from_rat(eps) < Real.from_rat(r2)
    a + Real.from_rat(eps) < b
}

theorem lt_add_right(a: Real, b: Real, c: Real) {
    a < b implies a + c < b + c
} by {
    let r: Rat satisfy {
        r.is_positive and a + Real.from_rat(r) < b
    }
    a + Real.from_rat(r) <= b
    (a + Real.from_rat(r)) + c <= b + c
    a + Real.from_rat(r) + c = a + c + Real.from_rat(r)
    a + c + Real.from_rat(r) <= b + c
    a + c < a + c + Real.from_rat(r)
    a + c < b + c
}

theorem lt_add_left(a: Real, b: Real, c: Real) {
    a < b implies c + a < c + b
}

theorem lt_add_cancel_right(a: Real, b: Real, c: Real) {
    a + c = b + c implies a = b
} by {
    not b < a
}

theorem lt_add_cancel_left(a: Real, b: Real, c: Real) {
    c + a = c + b implies a = b
}

// Whether -a > r. Which is just a < -r.
define neg_gt(a: Real, r: Rat) -> Bool {
    a < Real.from_rat(-r)
}

theorem lower_rat(a: Real) {
    exists(r: Rat) {
        Real.from_rat(r) < a
    }
}

theorem neg_gt_is_cut(a: Real) {
    is_cut(neg_gt(a))
} by {
    // Something not in the cut
    let r1: Rat satisfy {
        Real.from_rat(r1) < a
    }
    not neg_gt(a, -r1)

    // Something in the cut
    let r2: Rat satisfy {
        a < Real.from_rat(r2)
    }
}

theorem neg_gt_is_lower(a: Real) {
    is_lower(neg_gt(a))
} by {
    forall(x: Rat, y: Rat) {
        if neg_gt(a, y) and x < y {
            a < Real.from_rat(-y)
            -y < -x
            Real.from_rat(-y) < Real.from_rat(-x)
            a < Real.from_rat(-x)
            neg_gt(a, x)
        }
    }
}

theorem neg_gt_has_no_greatest(a: Real) {
    not has_greatest(neg_gt(a))
} by {
    if has_greatest(neg_gt(a)) {
        let q: Rat satisfy {
            is_greatest(neg_gt(a), q)
        }
        a < Real.from_rat(-q)
        let r: Rat satisfy {
            r.is_positive and a + Real.from_rat(r) < Real.from_rat(-q)
        }
        a + Real.from_rat(r) + Real.from_rat(-r) < Real.from_rat(-q) + Real.from_rat(-r)
        a + Real.from_rat(r) + Real.from_rat(-r) = a + Real.from_rat(r + -r)
        a + Real.from_rat(r + -r) < Real.from_rat(-q) + Real.from_rat(-r)
        q < q + r
        false
    }
}

theorem neg_gt_is_dedekind_cut(a: Real) {
    is_dedekind_cut(neg_gt(a))
}

let neg(a: Real) -> b: Real satisfy {
    Real.new(neg_gt((a))) = Option.some(b)
}

/// The negative of this real number.
instance Real: Neg {
    let neg = neg
}

theorem lt_rat_neg(a: Real, r: Rat) {
    a < Real.from_rat(-r) implies -a > Real.from_rat(r)
}

theorem neg_from_rat(r: Rat) {
    -Real.from_rat(r) = Real.from_rat(-r)
} by {
    if -Real.from_rat(r) > Real.from_rat(-r) {
        Real.from_rat(--r) > Real.from_rat(r)
        Real.from_rat(--r).gt_rat(r)
        false
    }
    if -Real.from_rat(r) < Real.from_rat(-r) {
        let q: Rat satisfy {
            -Real.from_rat(r) < Real.from_rat(q) and Real.from_rat(q) < Real.from_rat(-r)
        }
        q < -r
        r < -q
        false
    }
}

theorem neg_gt_rat(a: Real, r: Rat) {
    -a > Real.from_rat(r) implies a < Real.from_rat(-r)
}

theorem gt_rat_neg(a: Real, r: Rat) {
    a > Real.from_rat(-r) implies -a < Real.from_rat(r)
} by {
    // Use contrapositive of neg_gt_rat(-a, -r):
    // neg_gt_rat says: -x > Real.from_rat(s) implies x < Real.from_rat(-s)
    // With x = -a and s = -r: --a > Real.from_rat(-r) implies -a < Real.from_rat(--r) = -a < Real.from_rat(r)
    // We have a > Real.from_rat(-r), and we know --a = a eventually.
    // Let's use direct definition of negation.

    // -a < Real.from_rat(r) iff there exists q with -a < Real.from_rat(q) < Real.from_rat(r)
    // Since a > Real.from_rat(-r), there's a q with Real.from_rat(-r) < Real.from_rat(q) < a
    let q: Rat satisfy {
        Real.from_rat(-r) < Real.from_rat(q) and Real.from_rat(q) < a
    }
    -r < q
    -q < r
    // neg_gt(a, q) is true because a > Real.from_rat(q), which means a > Real.from_rat(-(-q))
    // i.e., by lt_rat_neg contrapositive... hmm.
    // Actually: neg_gt(a, -q) = true iff a < Real.from_rat(--q) = Real.from_rat(q)
    // We have Real.from_rat(q) < a, not a < Real.from_rat(q).
    // So neg_gt(a, -q) is false, meaning -a is NOT > Real.from_rat(-q)
    // Combined with order properties, -a <= Real.from_rat(-q)

    // Use neg_gt_rat with substitution: -(-a) > Real.from_rat(-q) implies -a < Real.from_rat(--q)
    // We need to show -a < Real.from_rat(r). We have -q < r, so Real.from_rat(-q) < Real.from_rat(r).
    // If -a <= Real.from_rat(-q) and Real.from_rat(-q) < Real.from_rat(r), then -a < Real.from_rat(r).

    not neg_gt(a, -q)
    not (-a > Real.from_rat(-q))
    -a <= Real.from_rat(-q)
    Real.from_rat(-q) < Real.from_rat(r)
}

theorem neg_lt_rat(a: Real, r: Rat) {
    -a < Real.from_rat(r) implies a > Real.from_rat(-r)
} by {
    Real.from_rat(-r) != a
}

theorem lt_swap_neg(a: Real, b: Real) {
    a < b implies -b < -a
} by {
    let r: Rat satisfy {
        a < Real.from_rat(r) and Real.from_rat(r) < b
    }
    -Real.from_rat(r) < -a
    Real.from_rat(r) < b
    -b < Real.from_rat(-r)
    Real.from_rat(-r) = -Real.from_rat(r)
    -b < -Real.from_rat(r)
}

theorem lt_neg_swap_neg(a: Real, b: Real) {
    a < -b implies b < -a
} by {
    let r: Rat satisfy {
        a < Real.from_rat(r) and Real.from_rat(r) < -b
    }
    -Real.from_rat(r) < -a
    Real.from_rat(-r) = -Real.from_rat(r)
    b < Real.from_rat(-r)
    b < -Real.from_rat(r)
}

theorem neg_neg(a: Real) {
    -(-a) = a
} by {
    if -(-a) < a {
        false
    }
    -(-a) >= a
    a <= -(-a)
    if a < -(-a) {
        -a < -a
        false
    }
    not (a < -(-a))
}

theorem neg_lt_swap_neg(a: Real, b: Real) {
    -a < b implies -b < a
} by {
    -b < -(-a)
}

theorem neg_lt_neg_swap_neg(a: Real, b: Real) {
    -a < -b implies b < a
}

theorem neg_zero {
    -Real.0 = Real.0
}

theorem neg_pos_is_neg(a: Real) {
    a.is_positive implies (-a).is_negative
}

theorem neg_neg_is_pos(a: Real) {
    (-a).is_negative implies a.is_positive
}

theorem gt_add_neg(a: Real, b: Real) {
    b.is_negative implies a > a + b
}

theorem rat_window(a: Real, eps: Rat) {
    eps.is_positive implies exists(r: Rat) {
        Real.from_rat(r) < a and a < Real.from_rat(r + eps)
    }
} by {
    a < a + Real.from_rat(eps)
    let top: Rat satisfy {
        a < Real.from_rat(top) and Real.from_rat(top) < a + Real.from_rat(eps)
    }
    let r = top + -eps
    r + eps = top
    Real.from_rat(r) < a
}

theorem add_neg_lte_zero(a: Real) {
    a + -a <= Real.0
} by {
    if a + -a > Real.0 {
        let r: Rat satisfy {
            Real.0 < Real.from_rat(r) and Real.from_rat(r) < a + -a
        }
        Real.from_rat(Rat.0) < Real.from_rat(r)
        Rat.0 < r
        r.is_positive
        let b: Rat satisfy {
            Real.from_rat(b) < a and a < Real.from_rat(b + r)
        }
        a + -a < Real.from_rat(b + r + -b)
        false
    }
}

theorem add_neg_gte_zero(a: Real) {
    a + -a >= Real.0
} by {
    if a + -a < Real.0 {
        a + -a < Real.0 implies exists(r: Rat) {
            a + -a < Real.from_rat(r) and Real.from_rat(r) < Real.0
        }
        exists(r: Rat) {
            Real.from_rat(r) < Real.0 and a + -a < Real.from_rat(r)
        }
        let r: Rat satisfy {
            Real.from_rat(r) < Real.0 and a + -a < Real.from_rat(r)
        }
        (-r).is_positive
        let b: Rat satisfy {
            Real.from_rat(b) < a and a < Real.from_rat(b + -r)
        }
        a < -Real.from_rat(-b + r)
        a + -a > Real.from_rat(b) + Real.from_rat(-b + r)
        false
    }
}

theorem add_neg_eq_zero(a: Real) {
    a + -a = Real.0
} by {
    a + -a <= Real.0
}

theorem neg_distrib(a: Real, b: Real) {
    -(a + b) = -a + -b
} by {
    // Show (a + b) + (-a + -b) = 0
    a + -a = Real.0
    b + -b = Real.0
    (a + b) + (-a + -b) = a + b + -a + -b
    a + b + -a + -b = a + -a + b + -b
    a + -a + b + -b = Real.0 + (b + -b)
    Real.0 + (b + -b) = Real.0 + Real.0
    Real.0 + Real.0 = Real.0
    (a + b) + (-a + -b) = Real.0
    // So -a + -b is the additive inverse of a + b
    -(a + b) + (a + b) = Real.0
    -(a + b) + (a + b) + (-a + -b) = Real.0 + (-a + -b)
    -(a + b) + Real.0 = -(a + b)
    -(a + b) = -a + -b
}

// Additive algebraic structure.

from algebra.add_semigroup import AddSemigroup

instance Real: AddSemigroup

from algebra.add_comm_semigroup import AddCommSemigroup

instance Real: AddCommSemigroup

from algebra.add_monoid import AddMonoid

instance Real: AddMonoid

from algebra.add_comm_monoid import AddCommMonoid

instance Real: AddCommMonoid

from algebra.add_group import AddGroup

instance Real: AddGroup

from algebra.add_comm_group import AddCommGroup

instance Real: AddCommGroup

from algebra.add_ordered_group import AddLeftOrderedGroup, AddOrderedGroup

instance Real: AddLeftOrderedGroup
instance Real: AddOrderedGroup

attributes Real {
    /// The absolute value of this real number.
    define abs(self) -> Real {
        if self.is_negative {
            -self
        } else {
            self
        }
    }

    /// The sign of this real number as a unit value (`-1` for negative, `1` for non-negative).
    define unit_sign(self) -> Real {
        if self.is_negative {
            -Real.1
        } else {
            Real.1
        }
    }
}

theorem pos_imp_eq_abs(a: Real) {
    a.is_positive implies a = a.abs
}

theorem sub_cancels(a: Real, b: Real) {
    a + b - b = a
}

theorem sub_moves_sides(a: Real, b: Real, c: Real) {
    a + b = c implies a = c - b
}

theorem lte_abs(a: Real) {
    a <= a.abs
} by {
    if a.is_negative {
        a < Real.0
        Real.0 < -a
        a < -a
        a.abs = -a
        a <= a.abs
    } else {
        a.abs = a
        a <= a.abs
    }
}

theorem abs_neg(a: Real) {
    (-a).abs = a.abs
} by {
    if a.is_negative {
    } else {
        if a = Real.0 {
            (-a).abs = a.abs
        } else {
            (-a).abs = a.abs
        }
    }
}

theorem abs_not_neg(a: Real) {
    not a.abs.is_negative
} by {
    if a.is_negative {
        a < Real.0
        -a > Real.0
        a.abs = -a
        a.abs.is_positive
        not a.abs.is_negative
    } else {
        a.abs = a
        not a.abs.is_negative
    }
}

theorem min_pos_pos(a: Real, b: Real) {
    a.is_positive and b.is_positive
    implies
    a.min(b).is_positive
} by {
    if a < b {
        min_eq_left_of_lt(a, b)
        a.min(b) = a
        a.min(b).is_positive
    } else {
        min_eq_right_of_not_lt(a, b)
        a.min(b) = b
        a.min(b).is_positive
    }
}

theorem max_pos_pos(a: Real, b: Real) {
    a.is_positive and b.is_positive
    implies
    a.max(b).is_positive
} by {
    if a > b {
        max_eq_left_of_gt(a, b)
        a.max(b) = a
        a.max(b).is_positive
    } else {
        max_eq_right_of_not_gt(a, b)
        a.max(b) = b
        a.max(b).is_positive
    }
}

theorem min_lte_left(a: Real, b: Real) {
    a.min(b) <= a
} by {
    if a < b {
    } else {
    }
}

theorem max_gte_left(a: Real, b: Real) {
    a.max(b) >= a
} by {
    if a > b {
    } else {
    }
}

theorem min_lte_right(a: Real, b: Real) {
    a.min(b) <= b
} by {
    if a < b {
    } else {
    }
}

theorem max_gte_right(a: Real, b: Real) {
    a.max(b) >= b
} by {
    if a > b {
    } else {
    }
}

theorem lt_both_imp_lt_min(a: Real, b: Real, c: Real) {
    a < b and a < c implies a < b.min(c)
} by {
    if a < b and a < c {
        lt_min_iff(a, b, c)
        a < b.min(c)
    }
}

theorem lt_min_imp_lt_left(a: Real, b: Real, c: Real) {
    a < b.min(c) implies a < b
}

theorem lt_min_imp_lt_right(a: Real, b: Real, c: Real) {
    a < b.min(c) implies a < c
}

theorem gt_both_imp_gt_max(a: Real, b: Real, c: Real) {
    a > b and a > c implies a > b.max(c)
} by {
    if a > b and a > c {
        a > b = b < a
        a > c = c < a
        max_lt_iff(b, c, a)
        b.max(c) < a
        a > b.max(c)
    }
}

theorem gt_max_imp_gt_left(a: Real, b: Real, c: Real) {
    a > b.max(c) implies a > b
}

theorem gt_max_imp_gt_right(a: Real, b: Real, c: Real) {
    a > b.max(c) implies a > c
}

instance Real: Meet {
    define meet(self, other: Real) -> Real {
        self.min(other)
    }
}

instance Real: Join {
    define join(self, other: Real) -> Real {
        self.max(other)
    }
}

theorem real_meet_lte_left(a: Real, b: Real) {
    a.meet(b) <= a
} by {
    min_lte_left(a, b)
}

theorem real_meet_lte_right(a: Real, b: Real) {
    a.meet(b) <= b
} by {
    min_lte_right(a, b)
}

theorem real_lte_meet_of_bounds(c: Real, a: Real, b: Real) {
    c <= a and c <= b implies c <= a.meet(b)
} by {
    if c <= a and c <= b {
        lte_min_of_bounds(c, a, b)
        c <= a.meet(b)
    }
}

instance Real: MeetSemilattice

theorem real_lte_join_left(a: Real, b: Real) {
    a <= a.join(b)
} by {
    lte_max_left(a, b)
}

theorem real_lte_join_right(a: Real, b: Real) {
    b <= a.join(b)
} by {
    lte_max_right(a, b)
}

theorem real_join_lte_of_bounds(a: Real, b: Real, c: Real) {
    a <= c and b <= c implies a.join(b) <= c
} by {
    if a <= c and b <= c {
        max_lte_of_upper_bounds(a, b, c)
        a.join(b) <= c
    }
}

instance Real: JoinSemilattice

instance Real: Lattice

theorem real_meet_join_distrib_left(a: Real, b: Real, c: Real) {
    a.meet(b.join(c)) = a.meet(b).join(a.meet(c))
} by {
    min_max_distrib_left(a, b, c)
}

theorem real_join_meet_distrib_left(a: Real, b: Real, c: Real) {
    a.join(b.meet(c)) = a.join(b).meet(a.join(c))
} by {
    max_min_distrib_left(a, b, c)
}

instance Real: DistribLattice

theorem rat_dual_upper_bound(a: Real, b: Real) {
    exists(r: Rat) {
        a < Real.from_rat(r) and b < Real.from_rat(r)
    }
} by {
    if a < b {
        let r: Rat satisfy {
            b < Real.from_rat(r)
        }
        a < Real.from_rat(r)
        a < Real.from_rat(r) and b < Real.from_rat(r)
        exists(s: Rat) {
            a < Real.from_rat(s) and b < Real.from_rat(s)
        }
    } else {
        b <= a
        let r: Rat satisfy {
            a < Real.from_rat(r)
        }
        b < Real.from_rat(r)
        a < Real.from_rat(r) and b < Real.from_rat(r)
        exists(s: Rat) {
            a < Real.from_rat(s) and b < Real.from_rat(s)
        }
    }
}

theorem abs_gte_zero(a: Real) {
    a.abs >= Real.0
} by {
    if a.is_negative {
    } else {
    }
}

theorem abs_from_rat(p: Rat) {
    Real.from_rat(p).abs = Real.from_rat(p.abs)
} by {
    if p.is_negative {
        Real.from_rat(p).is_negative
        Real.from_rat(p).abs = -Real.from_rat(p)
        -Real.from_rat(p) = Real.from_rat(-p)
        p.abs = -p
        Real.from_rat(p.abs) = Real.from_rat(-p)
        Real.from_rat(p).abs = Real.from_rat(p.abs)
    } else {
        not Real.from_rat(p).is_negative
        Real.from_rat(p).abs = Real.from_rat(p)
        p.abs = p
        Real.from_rat(p.abs) = Real.from_rat(p)
        Real.from_rat(p).abs = Real.from_rat(p.abs)
    }
}

attributes Real {
    /// True if this real number is within `eps` of the other real number.
    define is_close(self, other: Real, eps: Real) -> Bool {
        (self - other).abs < eps
    }
}

theorem close_imp_eps_pos(a: Real, b: Real, eps: Real) {
    a.is_close(b, eps) implies eps.is_positive
} by {
    (a - b).abs < eps
    not (a - b).abs.is_negative
    (a - b).abs >= Real.0
    Real.0 <= (a - b).abs
    Real.0 < eps
}

theorem close_comm(a: Real, b: Real, eps: Real) {
    a.is_close(b, eps) implies b.is_close(a, eps)
} by {
    (a - b).abs < eps
    b - a = -(a - b)
    (b - a).abs = (-(a - b)).abs
    (-(a - b)).abs = (a - b).abs
    (b - a).abs < eps
}

theorem close_imp_bounds(a: Real, b: Real, eps: Real) {
    a.is_close(b, eps) implies b - eps < a and a < b + eps and a > b - eps and b < a + eps
} by {
    // The premise is (a - b).abs < eps
    (a - b).abs < eps

    // Case 1: a - b is non-negative, so a - b < eps
    // Case 2: a - b is negative, so -(a - b) < eps, i.e., b - a < eps
    a - b < eps
    b - a < eps

    // From a - b < eps, adding b to both sides: a < eps + b = b + eps
    a - b + b = a
    a - b + b < eps + b
    a < eps + b
    eps + b = b + eps
    a < b + eps

    // From b - a < eps, adding a: b < eps + a = a + eps
    b - a + a = b
    b - a + a < eps + a
    b < eps + a
    eps + a = a + eps
    b < a + eps

    // For b - eps < a, we have a - b > -eps, so a > b - eps
    // From b - a < eps, negate: -(b - a) > -eps, i.e., a - b > -eps
    -(b - a) = a - b
    -(b - a) > -eps
    a - b > -eps
    a + -b > -eps
    a + -b + b > -eps + b
    a > -eps + b
    -eps + b = b - eps
    a > b - eps
    b - eps < a
}

theorem bounds_imp_close(a: Real, b: Real, eps: Real) {
    b - eps < a and a < b + eps implies a.is_close(b, eps)
} by {
    // Need to show (a - b).abs < eps
    if (a - b).is_negative {
        // a - b < 0, so abs(a-b) = -(a-b) = b - a
        (a - b).abs = -(a - b)
        -(a - b) = b - a
        (a - b).abs = b - a
        // From b - eps < a: b - a < eps
        b - eps + eps < a + eps
        b - eps + eps = b
        b < a + eps
        b + -a < a + eps + -a
        b - a < eps
        (a - b).abs < eps
    } else {
        // a - b >= 0, so abs(a-b) = a - b
        (a - b).abs = a - b
        // From a < b + eps: a - b < eps
        a + -b < b + eps + -b
        a - b < eps
        (a - b).abs < eps
    }
}

theorem sum_bounds_imp_close(a: Real, b: Real, eps: Real) {
    b < a + eps and a < b + eps implies a.is_close(b, eps)
} by {
    // From b < a + eps: b + -eps < a + eps + -eps = a
    b + -eps < a + eps + -eps
    a + eps + -eps = a
    b + -eps < a
    b - eps < a
}

theorem self_close(a: Real, eps: Real) {
    eps.is_positive implies a.is_close(a, eps)
}

// Closeness is preserved in the real<->rat conversion.
theorem close_rats_imp_close_reals(q: Rat, r: Rat, eps: Rat) {
    q.is_close(r, eps) implies Real.from_rat(q).is_close(Real.from_rat(r), Real.from_rat(eps))
} by {
    Real.from_rat((q - r).abs) < Real.from_rat(eps)
    (Real.from_rat(q) - Real.from_rat(r)).abs < Real.from_rat(eps)
}

theorem close_reals_imp_close_rats(q: Rat, r: Rat, eps: Rat) {
    Real.from_rat(q).is_close(Real.from_rat(r), Real.from_rat(eps))
    implies q.is_close(r, eps)
} by {
    // The premise is (Real.from_rat(q) - Real.from_rat(r)).abs < Real.from_rat(eps)
    (Real.from_rat(q) - Real.from_rat(r)).abs < Real.from_rat(eps)
    Real.from_rat(q) - Real.from_rat(r) = Real.from_rat(q - r)
    (Real.from_rat(q) - Real.from_rat(r)).abs = Real.from_rat(q - r).abs
    Real.from_rat(q - r).abs = Real.from_rat((q - r).abs)
    (Real.from_rat(q) - Real.from_rat(r)).abs = Real.from_rat((q - r).abs)
    Real.from_rat((q - r).abs) < Real.from_rat(eps)
    (q - r).abs < eps
}

// Every real can be approximated by a rational.
theorem rat_approx_exists(x: Real, eps: Rat) {
    eps.is_positive implies exists(r: Rat) {
        x.is_close(Real.from_rat(r), Real.from_rat(eps))
    }
} by {
    let r: Rat satisfy {
        Real.from_rat(r) < x and x < Real.from_rat(r + eps)
    }
}

// Every real has a rational that is a distant upper bound.
theorem rat_upper(x: Real, eps: Rat) {
    eps.is_positive implies exists(r: Rat) {
        not x.is_close(Real.from_rat(r), Real.from_rat(eps)) and x < Real.from_rat(r)
    }
} by {
    let r1: Rat satisfy {
        x < Real.from_rat(r1)
    }
    let r = r1 + eps
    if x.is_close(Real.from_rat(r), Real.from_rat(eps)) {
        x < Real.from_rat(r - eps)
        Real.from_rat(r) - Real.from_rat(eps) < x
        Real.from_rat(-eps) = -Real.from_rat(eps)
        Real.from_rat(r) + Real.from_rat(-eps) = Real.from_rat(r + -eps)
        Real.from_rat(r) - Real.from_rat(eps) = Real.from_rat(r + -eps)
        r + -eps = r - eps
        Real.from_rat(r) - Real.from_rat(eps) = Real.from_rat(r - eps)
        Real.from_rat(r - eps) < x
        x < Real.from_rat(r - eps) and Real.from_rat(r - eps) < x implies x < x
        false
    }
    x < Real.from_rat(r)
}

// Every real has a rational that is a distant lower bound.
theorem rat_lower(x: Real, eps: Rat) {
    eps.is_positive implies exists(r: Rat) {
        not x.is_close(Real.from_rat(r), Real.from_rat(eps)) and Real.from_rat(r) < x
    }
} by {
    let r1: Rat satisfy {
        Real.from_rat(r1) < x
    }
    let r = r1 - eps
    if x.is_close(Real.from_rat(r), Real.from_rat(eps)) {
        Real.from_rat(r + eps) < x
        false
    }
    r < r1
}

// Every intersecting pair of intervals has a rational number in it.
theorem rat_intersect(a: Real, b: Real, c: Real, d: Real, e: Real) {
    a.is_close(b, c) and a.is_close(d, e)
    implies
    exists(r: Rat) {
        Real.from_rat(r).is_close(b, c) and Real.from_rat(r).is_close(d, e)
    }
} by {
    a < (b + c).min(d + e)
    let r: Rat satisfy {
        a < Real.from_rat(r) and Real.from_rat(r) < (b + c).min(d + e)
    }
    Real.from_rat(r).is_close(d, e)
}

theorem swap_minus_plus(a: Real, b: Real, c: Real) {
    a + b - c = a - c + b
}
