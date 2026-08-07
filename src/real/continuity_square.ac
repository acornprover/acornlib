from data.basic.function_algebra import pointwise_mul
from data.basic.functions import identity_fn
from real.continuity_pointwise_mul import continuous_at_pointwise_mul,
    continuous_pointwise_mul
from real.continuity_composition import continuous_imp_continuous_at
from real.continuity_sequences import identity_function_is_continuous
from real.continuity_base import Real, continuous, continuous_at

/// The pointwise square of a real number.
define square_real(x: Real) -> Real {
    x * x
}

/// The pointwise square function agrees with the pointwise product of the identity with itself.
theorem square_real_eq_pointwise_mul_identity {
    square_real = pointwise_mul[Real, Real](identity_fn[Real], identity_fn[Real])
} by {
    forall(x: Real) {
        identity_fn[Real](x) = x
        pointwise_mul[Real, Real](identity_fn[Real], identity_fn[Real], x) = x * x
        square_real(x) = x * x
        square_real(x) = pointwise_mul[Real, Real](identity_fn[Real], identity_fn[Real], x)
    }
}

/// The square function on the reals is continuous at each point.
theorem continuous_at_square_real(x: Real) {
    continuous_at(square_real, x)
} by {
    identity_function_is_continuous
    continuous_imp_continuous_at(identity_fn[Real], x)
    continuous_at(identity_fn[Real], x)
    continuous_at_pointwise_mul(identity_fn[Real], identity_fn[Real], x)
    continuous_at(pointwise_mul[Real, Real](identity_fn[Real], identity_fn[Real]), x)
    square_real_eq_pointwise_mul_identity
    continuous_at(square_real, x)
}

/// The square function on the reals is continuous.
theorem continuous_square_real {
    continuous(square_real)
} by {
    identity_function_is_continuous
    continuous(identity_fn[Real])
    continuous_pointwise_mul(identity_fn[Real], identity_fn[Real])
    continuous(pointwise_mul[Real, Real](identity_fn[Real], identity_fn[Real]))
    square_real_eq_pointwise_mul_identity
    continuous(square_real)
}
