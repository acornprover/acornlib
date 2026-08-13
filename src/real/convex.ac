/// Convex functions on the real line.
///
/// A real function is convex on an interval when its graph lies below every
/// chord: for x <= y in the interval and t in [0, 1], the value at the convex
/// combination (1 - t) x + t y is at most the same convex combination of the
/// endpoint values.  This file proves the classical differentiability
/// characterization: a differentiable function is convex on an interval
/// exactly when its derivative is nondecreasing there.  The forward direction
/// combines the monotonicity of the chord slopes of a convex function with the
/// mean value theorem; the converse applies the mean value theorem to the two
/// halves of a chord.

from order import lte_refl, lte_trans, lte_antisymm, lt_trans, lt_imp_lte,
    lt_of_lt_of_lte, lt_of_lte_of_lt, not_lte_imp_gt, not_lt_imp_gte,
    lt_imp_ne, lt_imp_ne_symm, not_lt_self, lte_lt_trans,
    lt_of_lte_of_ne, lt_of_lte_of_ne_symm
from real.continuity_base import Real, continuous, continuous_at
from real.derivative_basic import has_derivative_at, has_derivative_at_delta,
    has_derivative_at_unique, difference_quotient, sub_ne_zero_of_ne
from real.calculus_api import is_derivative_fn, is_derivative_fn_at
from real.calculus_quotient_continuity_examples import is_derivative_fn_imp_continuous
from real.derivative_continuity import div_mul_cancel_denominator
from real.real_base import add_comm, add_assoc, add_zero_left, add_zero_right,
    add_neg_eq_zero, neg_distrib, neg_zero, neg_neg, sub_cancels, lt_add_pos,
    lt_add_right, lte_add_right, lt_add_converse, pos_gt_zero, gt_zero_imp_pos,
    pos_imp_eq_abs, close_imp_bounds, self_close, abs_gte_zero,
    lt_zero_imp_neg, neg_lt_zero, one_half_positive, one_half_plus_one_half,
    neg_lt_swap_neg, abs_neg, neg_pos_is_neg
from real.real_ring import mul_zero_left, mul_zero_right, real_mul_comm, mul_assoc,
    mul_one_left, mul_one_right, mul_distrib_right, mul_distrib_left,
    mul_neg_left, mul_neg_right, mul_sub_distrib_right, mul_sub_distrib_left
from real.real_field import mul_inverse, mul_left_cancel
from real.real_seq import eps_smaller_than_both, sub_zero_imp_eq, lt_imp_minus_pos,
    diff_pos_imp_lt
from real.mean_value import secant_slope, mean_value_theorem, lte_imp_sub_le_zero,
    lte_imp_zero_le_sub, le_zero_div_pos_le_zero, le_zero_div_neg_ge_zero
from ordered_field import inverse_of_positive_is_positive, mul_le_mul_of_nonpos_right,
    multiply_inequality_with_nonnegative_element, mul_le_mul_of_nonneg_right,
    mul_lt_mul_of_pos_right, mul_pos_pos, mul_right_scalar_map,
    mul_right_scalar_map_reflects_le_of_positive

numerals Real

/// True if f is convex on the interval [a, b]: the graph of f lies below every
/// chord of the interval.  Concretely, for x <= y in [a, b] and t in [0, 1],
/// the value at the convex combination (1 - t) x + t y is at most the same
/// convex combination of the endpoint values.
define convex_on(f: Real -> Real, a: Real, b: Real) -> Bool {
    forall(x: Real, y: Real, t: Real) {
        a <= x and x <= y and y <= b and Real.0 <= t and t <= Real.1
        implies f((Real.1 - t) * x + t * y) <= (Real.1 - t) * f(x) + t * f(y)
    }
}

/// Convexity instantiated at a single chord point.
theorem convex_on_apply(f: Real -> Real, a: Real, b: Real, x: Real, y: Real, t: Real) {
    convex_on(f, a, b) and a <= x and x <= y and y <= b and Real.0 <= t and t <= Real.1
    implies f((Real.1 - t) * x + t * y) <= (Real.1 - t) * f(x) + t * f(y)
} by {
    if convex_on(f, a, b) and a <= x and x <= y and y <= b and Real.0 <= t and t <= Real.1 {
        convex_on(f, a, b) = forall(x0: Real, y0: Real, t0: Real) {
            a <= x0 and x0 <= y0 and y0 <= b and Real.0 <= t0 and t0 <= Real.1
            implies f((Real.1 - t0) * x0 + t0 * y0) <= (Real.1 - t0) * f(x0) + t0 * f(y0)
        }
        f((Real.1 - t) * x + t * y) <= (Real.1 - t) * f(x) + t * f(y)
    }
}

/// One half is at most one.
theorem one_half_le_one {
    Real.one_half <= Real.1
} by {
    one_half_positive
    Real.0 < Real.one_half
    lt_imp_lte(Real.0, Real.one_half)
    Real.0 <= Real.one_half
    lte_add_right(Real.0, Real.one_half, Real.one_half)
    Real.0 + Real.one_half <= Real.one_half + Real.one_half
    add_zero_left(Real.one_half)
    Real.0 + Real.one_half = Real.one_half
    one_half_plus_one_half
    Real.one_half + Real.one_half = Real.1
    Real.one_half <= Real.1
}

/// One minus one half is one half.
theorem one_minus_half_eq_half {
    Real.1 - Real.one_half = Real.one_half
} by {
    one_half_plus_one_half
    Real.one_half + Real.one_half = Real.1
    Real.1 = Real.one_half + Real.one_half
    sub_cancels(Real.one_half, Real.one_half)
    Real.one_half + Real.one_half + -Real.one_half = Real.one_half
    Real.1 + -Real.one_half = Real.one_half
    Real.1 - Real.one_half = Real.1 + -Real.one_half
    Real.1 - Real.one_half = Real.one_half
}

/// A convex function satisfies Jensen's inequality at the midpoint: for x and
/// y in the interval, f((x + y)/2) <= (f(x) + f(y))/2.
theorem convex_midpoint_inequality(f: Real -> Real, a: Real, b: Real, x: Real, y: Real) {
    convex_on(f, a, b) and a <= x and x <= b and a <= y and y <= b
    implies f((x + y) * Real.one_half) <= (f(x) + f(y)) * Real.one_half
} by {
    if convex_on(f, a, b) and a <= x and x <= b and a <= y and y <= b {
        if x <= y {
            one_half_positive
            Real.0 < Real.one_half
            lt_imp_lte(Real.0, Real.one_half)
            Real.0 <= Real.one_half
            one_half_le_one
            Real.one_half <= Real.1
            convex_on_apply(f, a, b, x, y, Real.one_half)
            f((Real.1 - Real.one_half) * x + Real.one_half * y) <= (Real.1 - Real.one_half) * f(x) + Real.one_half * f(y)
            one_minus_half_eq_half
            Real.1 - Real.one_half = Real.one_half
            f(Real.one_half * x + Real.one_half * y) <= Real.one_half * f(x) + Real.one_half * f(y)
            mul_distrib_left(Real.one_half, x, y)
            Real.one_half * (x + y) = Real.one_half * x + Real.one_half * y
            real_mul_comm(Real.one_half, x + y)
            (x + y) * Real.one_half = Real.one_half * (x + y)
            f((x + y) * Real.one_half) <= Real.one_half * f(x) + Real.one_half * f(y)
            mul_distrib_left(Real.one_half, f(x), f(y))
            Real.one_half * (f(x) + f(y)) = Real.one_half * f(x) + Real.one_half * f(y)
            real_mul_comm(Real.one_half, f(x) + f(y))
            (f(x) + f(y)) * Real.one_half = Real.one_half * (f(x) + f(y))
            f((x + y) * Real.one_half) <= (f(x) + f(y)) * Real.one_half
        } else {
            not x <= y
            not_lte_imp_gt[Real](x, y)
            y < x
            lt_imp_lte(y, x)
            y <= x
            one_half_positive
            Real.0 < Real.one_half
            lt_imp_lte(Real.0, Real.one_half)
            Real.0 <= Real.one_half
            one_half_le_one
            Real.one_half <= Real.1
            convex_on_apply(f, a, b, y, x, Real.one_half)
            f((Real.1 - Real.one_half) * y + Real.one_half * x) <= (Real.1 - Real.one_half) * f(y) + Real.one_half * f(x)
            one_minus_half_eq_half
            Real.1 - Real.one_half = Real.one_half
            f(Real.one_half * y + Real.one_half * x) <= Real.one_half * f(y) + Real.one_half * f(x)
            mul_distrib_left(Real.one_half, y, x)
            Real.one_half * (y + x) = Real.one_half * y + Real.one_half * x
            add_comm(x, y)
            x + y = y + x
            real_mul_comm(Real.one_half, x + y)
            (x + y) * Real.one_half = Real.one_half * (x + y)
            f((x + y) * Real.one_half) <= Real.one_half * f(y) + Real.one_half * f(x)
            mul_distrib_left(Real.one_half, f(y), f(x))
            Real.one_half * (f(y) + f(x)) = Real.one_half * f(y) + Real.one_half * f(x)
            add_comm(f(x), f(y))
            f(x) + f(y) = f(y) + f(x)
            real_mul_comm(Real.one_half, f(x) + f(y))
            (f(x) + f(y)) * Real.one_half = Real.one_half * (f(x) + f(y))
            f((x + y) * Real.one_half) <= (f(x) + f(y)) * Real.one_half
        }
    }
}

/// A nonnegative numerator divided by a positive denominator is nonnegative.
theorem ge_zero_div_pos_ge_zero(a: Real, b: Real) {
    Real.0 <= a and b.is_positive implies Real.0 <= a / b
} by {
    if Real.0 <= a and b.is_positive {
        pos_gt_zero(b)
        b > Real.0
        inverse_of_positive_is_positive[Real](b)
        Real.0 < b.inverse
        lt_imp_lte(Real.0, b.inverse)
        Real.0 <= b.inverse
        multiply_inequality_with_nonnegative_element[Real](Real.0, a, b.inverse)
        Real.0 * b.inverse <= a * b.inverse
        mul_zero_left(b.inverse)
        Real.0 * b.inverse = Real.0
        Real.0 <= a * b.inverse
        a / b = a * b.inverse
        Real.0 <= a / b
    }
}

/// The inverse of a negation is the negation of the inverse.
theorem neg_inverse(a: Real) {
    a != Real.0 implies (-a).inverse = -a.inverse
} by {
    if a != Real.0 {
        if -a = Real.0 {
            neg_neg(a)
            -(-a) = a
            a = -Real.0
            neg_zero
            -Real.0 = Real.0
            a = Real.0
            false
        }
        -a != Real.0
        mul_inverse(-a)
        (-a) * (-a).inverse = Real.1
        mul_inverse(a)
        a * a.inverse = Real.1
        mul_neg_left(a, a.inverse)
        -a * a.inverse = -(a * a.inverse)
        mul_neg_right(-a, a.inverse)
        (-a) * -a.inverse = -((-a) * a.inverse)
        (-a) * -a.inverse = -(-(a * a.inverse))
        neg_neg(a * a.inverse)
        -(-(a * a.inverse)) = a * a.inverse
        (-a) * -a.inverse = a * a.inverse
        (-a) * -a.inverse = Real.1
        mul_left_cancel(Real.1, -a, (-a).inverse)
        Real.1 / (-a) = (-a).inverse
        mul_left_cancel(Real.1, -a, -a.inverse)
        Real.1 / (-a) = -a.inverse
        (-a).inverse = Real.1 / (-a)
        (-a).inverse = -a.inverse
    }
}

/// Negating both the numerator and the denominator leaves a quotient unchanged.
theorem neg_div_neg_eq(a: Real, b: Real) {
    b != Real.0 implies (-a) / (-b) = a / b
} by {
    if b != Real.0 {
        neg_inverse(b)
        (-b).inverse = -b.inverse
        (-a) / (-b) = (-a) * (-b).inverse
        (-a) * (-b).inverse = (-a) * -b.inverse
        mul_neg_right(-a, b.inverse)
        (-a) * -b.inverse = -((-a) * b.inverse)
        mul_neg_left(a, b.inverse)
        -a * b.inverse = -(a * b.inverse)
        (-a) * -b.inverse = a * b.inverse
        a * b.inverse = a / b
        (-a) / (-b) = a / b
    }
}

/// Subtraction cancels through a common minuend: (a - b) - (a - c) = c - b.
theorem sub_sub_cancel(a: Real, b: Real, c: Real) {
    (a - b) - (a - c) = c - b
} by {
    neg_distrib(a, -c)
    -(a + -c) = -a + -(-c)
    neg_neg(c)
    -(-c) = c
    -a + -(-c) = -a + c
    a - c = a + -c
    -(a - c) = -(a + -c)
    (a - b) - (a - c) = (a + -b) + -(a - c)
    (a + -b) + -(a - c) = (a + -b) + (-a + c)
    add_assoc(a, -b, -a + c)
    (a + -b) + (-a + c) = a + (-b + (-a + c))
    add_assoc(-b, -a, c)
    -b + (-a + c) = (-b + -a) + c
    add_comm(-b, -a)
    -b + -a = -a + -b
    (-b + -a) + c = (-a + -b) + c
    a + (-b + (-a + c)) = a + ((-a + -b) + c)
    add_assoc(a, -a + -b, c)
    a + ((-a + -b) + c) = (a + (-a + -b)) + c
    add_assoc(a, -a, -b)
    (a + -a) + -b = a + (-a + -b)
    add_neg_eq_zero(a)
    a + -a = Real.0
    ((a + -a) + -b) + c = (Real.0 + -b) + c
    add_zero_left(-b)
    Real.0 + -b = -b
    (Real.0 + -b) + c = -b + c
    c - b = c + -b
    (a - b) - (a - c) = c - b
}

/// Subtracting a difference cancels: a - (a - b) = b.
theorem sub_minus_sub(a: Real, b: Real) {
    a - (a - b) = b
} by {
    sub_sub_cancel(a, Real.0, b)
    (a - Real.0) - (a - b) = b - Real.0
    neg_zero
    -Real.0 = Real.0
    a - Real.0 = a + -Real.0
    a + -Real.0 = a + Real.0
    add_zero_right(a)
    a + Real.0 = a
    a - Real.0 = a
    b - Real.0 = b + -Real.0
    b + -Real.0 = b + Real.0
    add_zero_right(b)
    b + Real.0 = b
    b - Real.0 = b
    a - (a - b) = b
}

/// For t in [0, 1], both 1 - t and t lie in [0, 1].
theorem one_minus_t_bounds(t: Real) {
    Real.0 <= t and t <= Real.1 implies Real.0 <= Real.1 - t and Real.1 - t <= Real.1
} by {
    if Real.0 <= t and t <= Real.1 {
        lte_add_right(t, Real.1, -t)
        t + -t <= Real.1 + -t
        add_neg_eq_zero(t)
        t + -t = Real.0
        Real.0 <= Real.1 + -t
        Real.1 - t = Real.1 + -t
        Real.0 <= Real.1 - t
        lte_add_right(Real.0, t, Real.1)
        Real.0 + Real.1 <= t + Real.1
        add_zero_left(Real.1)
        Real.0 + Real.1 = Real.1
        Real.1 <= t + Real.1
        lte_add_right(Real.1, t + Real.1, -t)
        Real.1 + -t <= (t + Real.1) + -t
        add_comm(t, Real.1)
        t + Real.1 = Real.1 + t
        (t + Real.1) + -t = (Real.1 + t) + -t
        add_assoc(Real.1, t, -t)
        (Real.1 + t) + -t = Real.1 + (t + -t)
        add_neg_eq_zero(t)
        t + -t = Real.0
        Real.1 + (t + -t) = Real.1 + Real.0
        add_zero_right(Real.1)
        Real.1 + Real.0 = Real.1
        (Real.1 + t) + -t = Real.1
        (t + Real.1) + -t = Real.1
        Real.1 + -t <= Real.1
        Real.1 - t = Real.1 + -t
        Real.1 - t <= Real.1
        Real.0 <= Real.1 - t and Real.1 - t <= Real.1
    }
}

/// A convex function satisfies the two-point weighted inequality for every
/// weight: for x and y in the interval and t in [0, 1],
/// f(t * x + (1 - t) * y) <= t * f(x) + (1 - t) * f(y).  Unlike the defining
/// chord condition, the endpoints may appear in either order.
theorem convex_weighted_inequality(f: Real -> Real, a: Real, b: Real, x: Real, y: Real, t: Real) {
    convex_on(f, a, b) and a <= x and x <= b and a <= y and y <= b and Real.0 <= t and t <= Real.1
    implies f(t * x + (Real.1 - t) * y) <= t * f(x) + (Real.1 - t) * f(y)
} by {
    if convex_on(f, a, b) and a <= x and x <= b and a <= y and y <= b and Real.0 <= t and t <= Real.1 {
        one_minus_t_bounds(t)
        Real.0 <= Real.1 - t and Real.1 - t <= Real.1
        Real.0 <= Real.1 - t
        Real.1 - t <= Real.1
        if x <= y {
            convex_on_apply(f, a, b, x, y, Real.1 - t)
            f((Real.1 - (Real.1 - t)) * x + (Real.1 - t) * y) <= (Real.1 - (Real.1 - t)) * f(x) + (Real.1 - t) * f(y)
            sub_minus_sub(Real.1, t)
            Real.1 - (Real.1 - t) = t
            f(t * x + (Real.1 - t) * y) <= t * f(x) + (Real.1 - t) * f(y)
        } else {
            not x <= y
            not_lte_imp_gt[Real](x, y)
            y < x
            lt_imp_lte(y, x)
            y <= x
            convex_on_apply(f, a, b, y, x, t)
            f((Real.1 - t) * y + t * x) <= (Real.1 - t) * f(y) + t * f(x)
            add_comm(t * x, (Real.1 - t) * y)
            t * x + (Real.1 - t) * y = (Real.1 - t) * y + t * x
            f((Real.1 - t) * y + t * x) = f(t * x + (Real.1 - t) * y)
            f(t * x + (Real.1 - t) * y) <= (Real.1 - t) * f(y) + t * f(x)
            add_comm(t * f(x), (Real.1 - t) * f(y))
            t * f(x) + (Real.1 - t) * f(y) = (Real.1 - t) * f(y) + t * f(x)
            f(t * x + (Real.1 - t) * y) <= t * f(x) + (Real.1 - t) * f(y)
        }
    }
}

/// The difference quotient at a point is the secant slope through the same pair.
theorem difference_quotient_eq_secant_slope(f: Real -> Real, x0: Real, x: Real) {
    x != x0 implies difference_quotient(f, x0, x) = secant_slope(f, x0, x)
} by {
    if x != x0 {
        difference_quotient(f, x0, x) = (f(x) - f(x0)) / (x - x0)
        secant_slope(f, x0, x) = (f(x) - f(x0)) / (x - x0)
        difference_quotient(f, x0, x) = secant_slope(f, x0, x)
    }
}

/// The difference quotient at a point is the secant slope through the pair in
/// the opposite order.
theorem difference_quotient_swap_eq_secant_slope(f: Real -> Real, x0: Real, x: Real) {
    x != x0 implies difference_quotient(f, x0, x) = secant_slope(f, x, x0)
} by {
    if x != x0 {
        difference_quotient(f, x0, x) = (f(x) - f(x0)) / (x - x0)
        sub_ne_zero_of_ne(x, x0)
        x - x0 != Real.0
        neg_div_neg_eq(f(x) - f(x0), x - x0)
        (-(f(x) - f(x0))) / (-(x - x0)) = (f(x) - f(x0)) / (x - x0)
        neg_distrib(f(x), f(x0))
        -(f(x) - f(x0)) = -f(x) + f(x0)
        add_comm(-f(x), f(x0))
        -f(x) + f(x0) = f(x0) + -f(x)
        f(x0) - f(x) = f(x0) + -f(x)
        -(f(x) - f(x0)) = f(x0) - f(x)
        neg_distrib(x, -x0)
        -(x + -x0) = -x + -(-x0)
        neg_neg(x0)
        -(-x0) = x0
        -x + -(-x0) = -x + x0
        add_comm(-x, x0)
        -x + x0 = x0 + -x
        x0 - x = x0 + -x
        -(x + -x0) = x0 - x
        x - x0 = x + -x0
        -(x - x0) = x0 - x
        (-(f(x) - f(x0))) / (-(x - x0)) = (f(x0) - f(x)) / (x0 - x)
        (f(x0) - f(x)) / (x0 - x) = (f(x) - f(x0)) / (x - x0)
        secant_slope(f, x, x0) = (f(x0) - f(x)) / (x0 - x)
        difference_quotient(f, x0, x) = secant_slope(f, x, x0)
    }
}

/// The chord slopes of a convex function are monotone in the second argument:
/// for x < z < w in the interval, slope(x, z) <= slope(x, w).
/// Convexity applied at the interior chord point z between x and w: with the
/// weight t = (z - x) / (w - x) the chord inequality
/// f(z) <= (1 - t) f(x) + t f(w) holds.
theorem convex_chord_combination(f: Real -> Real, a: Real, b: Real, x: Real, z: Real, w: Real) {
    convex_on(f, a, b) and a <= x and x < z and z < w and w <= b
    implies f(z) <= (Real.1 - (z - x) * (w - x).inverse) * f(x) + (z - x) * (w - x).inverse * f(w)
} by {
    if convex_on(f, a, b) and a <= x and x < z and z < w and w <= b {
        lt_imp_minus_pos(x, z)
        (z - x).is_positive
        lt_trans[Real](x, z, w)
        x < w
        lt_imp_minus_pos(x, w)
        (w - x).is_positive
        pos_gt_zero(z - x)
        z - x > Real.0
        pos_gt_zero(w - x)
        w - x > Real.0
        inverse_of_positive_is_positive[Real](w - x)
        Real.0 < (w - x).inverse
        mul_pos_pos(z - x, (w - x).inverse)
        Real.0 < (z - x) * (w - x).inverse
        lt_imp_lte(Real.0, (z - x) * (w - x).inverse)
        Real.0 <= (z - x) * (w - x).inverse
        lt_add_right(z, w, -x)
        z + -x < w + -x
        z - x = z + -x
        w - x = w + -x
        z - x < w - x
        mul_lt_mul_of_pos_right[Real](z - x, w - x, (w - x).inverse)
        (z - x) * (w - x).inverse < (w - x) * (w - x).inverse
        lt_imp_ne_symm(Real.0, w - x)
        (w - x) != Real.0
        mul_inverse(w - x)
        (w - x) * (w - x).inverse = Real.1
        (z - x) * (w - x).inverse < Real.1
        lt_imp_lte((z - x) * (w - x).inverse, Real.1)
        (z - x) * (w - x).inverse <= Real.1
        // z = (1 - t) x + t w with t = (z - x) * (w - x).inverse
        mul_sub_distrib_left(Real.1, (z - x) * (w - x).inverse, x)
        (Real.1 - (z - x) * (w - x).inverse) * x =
            Real.1 * x - (z - x) * (w - x).inverse * x
        mul_one_left(x)
        Real.1 * x = x
        (Real.1 - (z - x) * (w - x).inverse) * x = x - (z - x) * (w - x).inverse * x
        add_assoc(x, -((z - x) * (w - x).inverse * x), (z - x) * (w - x).inverse * w)
        (x + -((z - x) * (w - x).inverse * x)) + (z - x) * (w - x).inverse * w =
            x + (-((z - x) * (w - x).inverse * x) + (z - x) * (w - x).inverse * w)
        add_comm(-((z - x) * (w - x).inverse * x), (z - x) * (w - x).inverse * w)
        -((z - x) * (w - x).inverse * x) + (z - x) * (w - x).inverse * w =
            (z - x) * (w - x).inverse * w + -((z - x) * (w - x).inverse * x)
        x + (-((z - x) * (w - x).inverse * x) + (z - x) * (w - x).inverse * w) =
            x + ((z - x) * (w - x).inverse * w - (z - x) * (w - x).inverse * x)
        (Real.1 - (z - x) * (w - x).inverse) * x + (z - x) * (w - x).inverse * w =
            (x + -((z - x) * (w - x).inverse * x)) + (z - x) * (w - x).inverse * w
        (Real.1 - (z - x) * (w - x).inverse) * x + (z - x) * (w - x).inverse * w =
            x + ((z - x) * (w - x).inverse * w - (z - x) * (w - x).inverse * x)
        mul_sub_distrib_right((z - x) * (w - x).inverse, w, x)
        (z - x) * (w - x).inverse * (w - x) =
            (z - x) * (w - x).inverse * w - (z - x) * (w - x).inverse * x
        (Real.1 - (z - x) * (w - x).inverse) * x + (z - x) * (w - x).inverse * w =
            x + (z - x) * (w - x).inverse * (w - x)
        mul_assoc(z - x, (w - x).inverse, w - x)
        (z - x) * ((w - x).inverse * (w - x)) = (z - x) * (w - x).inverse * (w - x)
        real_mul_comm((w - x).inverse, w - x)
        (w - x).inverse * (w - x) = (w - x) * (w - x).inverse
        (w - x) * (w - x).inverse = Real.1
        (z - x) * ((w - x).inverse * (w - x)) = (z - x) * Real.1
        mul_one_right(z - x)
        (z - x) * Real.1 = z - x
        (z - x) * (w - x).inverse * (w - x) = z - x
        (Real.1 - (z - x) * (w - x).inverse) * x + (z - x) * (w - x).inverse * w = x + (z - x)
        add_comm(x, z)
        x + z = z + x
        sub_cancels(z, x)
        z + x + -x = z
        (x + z) + -x = (z + x) + -x
        x + (z - x) = x + (z + -x)
        x + (z + -x) = (x + z) + -x
        x + (z - x) = z
        (Real.1 - (z - x) * (w - x).inverse) * x + (z - x) * (w - x).inverse * w = z
        // convexity at (x, w) with this t
        lt_imp_lte(x, w)
        x <= w
        convex_on_apply(f, a, b, x, w, (z - x) * (w - x).inverse)
        (f((Real.1 - (z - x) * (w - x).inverse) * x + (z - x) * (w - x).inverse * w)
            <= (Real.1 - (z - x) * (w - x).inverse) * f(x) + (z - x) * (w - x).inverse * f(w))
        f((Real.1 - (z - x) * (w - x).inverse) * x + (z - x) * (w - x).inverse * w) = f(z)
        f(z) <= (Real.1 - (z - x) * (w - x).inverse) * f(x) + (z - x) * (w - x).inverse * f(w)
    }
}

/// The chord slopes of a convex function are monotone in the second argument:
/// for x < z < w in the interval, slope(x, z) <= slope(x, w).
theorem convex_slope_second_arg_monotone(f: Real -> Real, a: Real, b: Real, x: Real, z: Real, w: Real) {
    convex_on(f, a, b) and a <= x and x < z and z < w and w <= b
    implies secant_slope(f, x, z) <= secant_slope(f, x, w)
} by {
    if convex_on(f, a, b) and a <= x and x < z and z < w and w <= b {
        convex_chord_combination(f, a, b, x, z, w)
        f(z) <= (Real.1 - (z - x) * (w - x).inverse) * f(x) + (z - x) * (w - x).inverse * f(w)
        // subtract f(x): f(z) - f(x) <= t (f(w) - f(x))
        lte_add_right(f(z), (Real.1 - (z - x) * (w - x).inverse) * f(x) + (z - x) * (w - x).inverse * f(w), -f(x))
        f(z) + -f(x) <= ((Real.1 - (z - x) * (w - x).inverse) * f(x) + (z - x) * (w - x).inverse * f(w)) + -f(x)
        f(z) - f(x) = f(z) + -f(x)
        f(z) - f(x) <= ((Real.1 - (z - x) * (w - x).inverse) * f(x) + (z - x) * (w - x).inverse * f(w)) + -f(x)
        mul_sub_distrib_left(Real.1, (z - x) * (w - x).inverse, f(x))
        (Real.1 - (z - x) * (w - x).inverse) * f(x) =
            Real.1 * f(x) - (z - x) * (w - x).inverse * f(x)
        mul_one_left(f(x))
        Real.1 * f(x) = f(x)
        (Real.1 - (z - x) * (w - x).inverse) * f(x) = f(x) - (z - x) * (w - x).inverse * f(x)
        add_assoc(f(x), -((z - x) * (w - x).inverse * f(x)), -f(x))
        (f(x) + -((z - x) * (w - x).inverse * f(x))) + -f(x) =
            f(x) + (-((z - x) * (w - x).inverse * f(x)) + -f(x))
        add_comm(-((z - x) * (w - x).inverse * f(x)), -f(x))
        -((z - x) * (w - x).inverse * f(x)) + -f(x) = -f(x) + -((z - x) * (w - x).inverse * f(x))
        add_assoc(f(x), -f(x), -((z - x) * (w - x).inverse * f(x)))
        (f(x) + -f(x)) + -((z - x) * (w - x).inverse * f(x)) =
            f(x) + (-f(x) + -((z - x) * (w - x).inverse * f(x)))
        add_neg_eq_zero(f(x))
        f(x) + -f(x) = Real.0
        (f(x) + -f(x)) + -((z - x) * (w - x).inverse * f(x)) =
            Real.0 + -((z - x) * (w - x).inverse * f(x))
        add_zero_left(-((z - x) * (w - x).inverse * f(x)))
        Real.0 + -((z - x) * (w - x).inverse * f(x)) = -((z - x) * (w - x).inverse * f(x))
        f(x) + (-f(x) + -((z - x) * (w - x).inverse * f(x))) = -((z - x) * (w - x).inverse * f(x))
        f(x) + (-((z - x) * (w - x).inverse * f(x)) + -f(x)) = -((z - x) * (w - x).inverse * f(x))
        (f(x) + -((z - x) * (w - x).inverse * f(x))) + -f(x) = -((z - x) * (w - x).inverse * f(x))
        add_assoc(f(x) + -((z - x) * (w - x).inverse * f(x)), (z - x) * (w - x).inverse * f(w), -f(x))
        ((f(x) + -((z - x) * (w - x).inverse * f(x))) + (z - x) * (w - x).inverse * f(w)) + -f(x) =
            (f(x) + -((z - x) * (w - x).inverse * f(x))) + ((z - x) * (w - x).inverse * f(w) + -f(x))
        add_comm((z - x) * (w - x).inverse * f(w), -f(x))
        (z - x) * (w - x).inverse * f(w) + -f(x) = -f(x) + (z - x) * (w - x).inverse * f(w)
        (f(x) + -((z - x) * (w - x).inverse * f(x))) + ((z - x) * (w - x).inverse * f(w) + -f(x)) =
            (f(x) + -((z - x) * (w - x).inverse * f(x))) + (-f(x) + (z - x) * (w - x).inverse * f(w))
        add_assoc(f(x) + -((z - x) * (w - x).inverse * f(x)), -f(x), (z - x) * (w - x).inverse * f(w))
        ((f(x) + -((z - x) * (w - x).inverse * f(x))) + -f(x)) + (z - x) * (w - x).inverse * f(w) =
            (f(x) + -((z - x) * (w - x).inverse * f(x))) + (-f(x) + (z - x) * (w - x).inverse * f(w))
        ((f(x) + -((z - x) * (w - x).inverse * f(x))) + -f(x)) + (z - x) * (w - x).inverse * f(w) =
            -((z - x) * (w - x).inverse * f(x)) + (z - x) * (w - x).inverse * f(w)
        ((f(x) + -((z - x) * (w - x).inverse * f(x))) + (z - x) * (w - x).inverse * f(w)) + -f(x) =
            -((z - x) * (w - x).inverse * f(x)) + (z - x) * (w - x).inverse * f(w)
        (f(x) - (z - x) * (w - x).inverse * f(x)) + (z - x) * (w - x).inverse * f(w) + -f(x) =
            -((z - x) * (w - x).inverse * f(x)) + (z - x) * (w - x).inverse * f(w)
        add_comm(-((z - x) * (w - x).inverse * f(x)), (z - x) * (w - x).inverse * f(w))
        -((z - x) * (w - x).inverse * f(x)) + (z - x) * (w - x).inverse * f(w) =
            (z - x) * (w - x).inverse * f(w) + -((z - x) * (w - x).inverse * f(x))
        (f(x) - (z - x) * (w - x).inverse * f(x)) + (z - x) * (w - x).inverse * f(w) + -f(x) =
            (z - x) * (w - x).inverse * f(w) - (z - x) * (w - x).inverse * f(x)
        mul_sub_distrib_right((z - x) * (w - x).inverse, f(w), f(x))
        (z - x) * (w - x).inverse * (f(w) - f(x)) =
            (z - x) * (w - x).inverse * f(w) - (z - x) * (w - x).inverse * f(x)
        ((Real.1 - (z - x) * (w - x).inverse) * f(x) + (z - x) * (w - x).inverse * f(w)) + -f(x) =
            (z - x) * (w - x).inverse * (f(w) - f(x))
        f(z) - f(x) <= (z - x) * (w - x).inverse * (f(w) - f(x))
        // divide by (z - x)
        lt_imp_minus_pos(x, z)
        (z - x).is_positive
        pos_gt_zero(z - x)
        z - x > Real.0
        inverse_of_positive_is_positive[Real](z - x)
        Real.0 < (z - x).inverse
        lt_imp_lte(Real.0, (z - x).inverse)
        Real.0 <= (z - x).inverse
        mul_le_mul_of_nonneg_right[Real](f(z) - f(x), (z - x) * (w - x).inverse * (f(w) - f(x)), (z - x).inverse)
        (f(z) - f(x)) * (z - x).inverse <= ((z - x) * (w - x).inverse * (f(w) - f(x))) * (z - x).inverse
        (f(z) - f(x)) * (z - x).inverse = (f(z) - f(x)) / (z - x)
        (f(z) - f(x)) / (z - x) <= ((z - x) * (w - x).inverse * (f(w) - f(x))) * (z - x).inverse
        mul_assoc(z - x, (w - x).inverse, f(w) - f(x))
        (z - x) * ((w - x).inverse * (f(w) - f(x))) = (z - x) * (w - x).inverse * (f(w) - f(x))
        real_mul_comm((w - x).inverse * (f(w) - f(x)), (z - x).inverse)
        ((w - x).inverse * (f(w) - f(x))) * (z - x).inverse = (z - x).inverse * ((w - x).inverse * (f(w) - f(x)))
        (z - x) * (((w - x).inverse * (f(w) - f(x))) * (z - x).inverse) =
            (z - x) * ((z - x).inverse * ((w - x).inverse * (f(w) - f(x))))
        mul_assoc(z - x, (z - x).inverse, (w - x).inverse * (f(w) - f(x)))
        (z - x) * ((z - x).inverse * ((w - x).inverse * (f(w) - f(x)))) =
            ((z - x) * (z - x).inverse) * ((w - x).inverse * (f(w) - f(x)))
        lt_imp_ne_symm(Real.0, z - x)
        (z - x) != Real.0
        mul_inverse(z - x)
        (z - x) * (z - x).inverse = Real.1
        ((z - x) * (z - x).inverse) * ((w - x).inverse * (f(w) - f(x))) =
            Real.1 * ((w - x).inverse * (f(w) - f(x)))
        mul_one_left((w - x).inverse * (f(w) - f(x)))
        Real.1 * ((w - x).inverse * (f(w) - f(x))) = (w - x).inverse * (f(w) - f(x))
        real_mul_comm((w - x).inverse, f(w) - f(x))
        (w - x).inverse * (f(w) - f(x)) = (f(w) - f(x)) * (w - x).inverse
        ((z - x) * (w - x).inverse * (f(w) - f(x))) * (z - x).inverse =
            (f(w) - f(x)) * (w - x).inverse
        (f(z) - f(x)) / (z - x) <= (f(w) - f(x)) * (w - x).inverse
        (f(w) - f(x)) * (w - x).inverse = (f(w) - f(x)) / (w - x)
        (f(z) - f(x)) / (z - x) <= (f(w) - f(x)) / (w - x)
        secant_slope(f, x, z) = (f(z) - f(x)) / (z - x)
        secant_slope(f, x, w) = (f(w) - f(x)) / (w - x)
        secant_slope(f, x, z) <= secant_slope(f, x, w)
    }
}

/// The chord slopes of a convex function are monotone in the first argument:
/// for x < z < w in the interval, slope(x, w) <= slope(z, w).
theorem convex_slope_first_arg_monotone(f: Real -> Real, a: Real, b: Real, x: Real, z: Real, w: Real) {
    convex_on(f, a, b) and a <= x and x < z and z < w and w <= b
    implies secant_slope(f, x, w) <= secant_slope(f, z, w)
} by {
    if convex_on(f, a, b) and a <= x and x < z and z < w and w <= b {
        convex_chord_combination(f, a, b, x, z, w)
        f(z) <= (Real.1 - (z - x) * (w - x).inverse) * f(x) + (z - x) * (w - x).inverse * f(w)
        // (w - x) is positive
        lt_trans[Real](x, z, w)
        x < w
        lt_imp_minus_pos(x, w)
        (w - x).is_positive
        pos_gt_zero(w - x)
        w - x > Real.0
        // multiply the convexity inequality by (w - x)
        mul_le_mul_of_nonneg_right[Real](f(z), (Real.1 - (z - x) * (w - x).inverse) * f(x) + (z - x) * (w - x).inverse * f(w), w - x)
        f(z) * (w - x) <= ((Real.1 - (z - x) * (w - x).inverse) * f(x) + (z - x) * (w - x).inverse * f(w)) * (w - x)
        // expand the right side
        mul_distrib_left((Real.1 - (z - x) * (w - x).inverse) * f(x), (z - x) * (w - x).inverse * f(w), w - x)
        ((Real.1 - (z - x) * (w - x).inverse) * f(x) + (z - x) * (w - x).inverse * f(w)) * (w - x) =
            (Real.1 - (z - x) * (w - x).inverse) * f(x) * (w - x) + (z - x) * (w - x).inverse * f(w) * (w - x)
        // (1 - t) f(x) (w - x) = f(x) (w - z)
        real_mul_comm(Real.1 - (z - x) * (w - x).inverse, f(x))
        (Real.1 - (z - x) * (w - x).inverse) * f(x) = f(x) * (Real.1 - (z - x) * (w - x).inverse)
        (Real.1 - (z - x) * (w - x).inverse) * f(x) * (w - x) =
            f(x) * ((Real.1 - (z - x) * (w - x).inverse) * (w - x))
        mul_sub_distrib_left(Real.1, (z - x) * (w - x).inverse, w - x)
        (Real.1 - (z - x) * (w - x).inverse) * (w - x) =
            Real.1 * (w - x) - (z - x) * (w - x).inverse * (w - x)
        mul_one_left(w - x)
        Real.1 * (w - x) = w - x
        // t (w - x) = z - x
        mul_assoc(z - x, (w - x).inverse, w - x)
        (z - x) * ((w - x).inverse * (w - x)) = (z - x) * (w - x).inverse * (w - x)
        real_mul_comm((w - x).inverse, w - x)
        (w - x).inverse * (w - x) = (w - x) * (w - x).inverse
        lt_imp_ne_symm(Real.0, w - x)
        (w - x) != Real.0
        mul_inverse(w - x)
        (w - x) * (w - x).inverse = Real.1
        (z - x) * ((w - x).inverse * (w - x)) = (z - x) * Real.1
        mul_one_right(z - x)
        (z - x) * Real.1 = z - x
        (z - x) * (w - x).inverse * (w - x) = z - x
        (Real.1 - (z - x) * (w - x).inverse) * (w - x) = (w - x) - (z - x)
        (w - x) - (z - x) = w - z
        (Real.1 - (z - x) * (w - x).inverse) * (w - x) = w - z
        (Real.1 - (z - x) * (w - x).inverse) * f(x) * (w - x) = f(x) * (w - z)
        // t f(w) (w - x) = f(w) (z - x)
        real_mul_comm((z - x) * (w - x).inverse, f(w))
        (z - x) * (w - x).inverse * f(w) = f(w) * ((z - x) * (w - x).inverse)
        (z - x) * (w - x).inverse * f(w) * (w - x) =
            f(w) * ((z - x) * (w - x).inverse) * (w - x)
        mul_assoc(f(w), (z - x) * (w - x).inverse, w - x)
        f(w) * (((z - x) * (w - x).inverse) * (w - x)) =
            f(w) * ((z - x) * (w - x).inverse) * (w - x)
        (z - x) * (w - x).inverse * (w - x) = z - x
        f(w) * (((z - x) * (w - x).inverse) * (w - x)) = f(w) * (z - x)
        (z - x) * (w - x).inverse * f(w) * (w - x) = f(w) * (z - x)
        ((Real.1 - (z - x) * (w - x).inverse) * f(x) + (z - x) * (w - x).inverse * f(w)) * (w - x) =
            f(x) * (w - z) + f(w) * (z - x)
        f(z) * (w - x) <= f(x) * (w - z) + f(w) * (z - x)
        // rearrange: f(z) (w - x) - f(x) (w - z) <= f(w) (z - x)
        lte_add_right(f(z) * (w - x), f(x) * (w - z) + f(w) * (z - x), -(f(x) * (w - z)))
        f(z) * (w - x) + -(f(x) * (w - z)) <= (f(x) * (w - z) + f(w) * (z - x)) + -(f(x) * (w - z))
        add_comm(f(x) * (w - z), f(w) * (z - x))
        f(x) * (w - z) + f(w) * (z - x) = f(w) * (z - x) + f(x) * (w - z)
        sub_cancels(f(w) * (z - x), f(x) * (w - z))
        (f(w) * (z - x) + f(x) * (w - z)) + -(f(x) * (w - z)) = f(w) * (z - x)
        (f(x) * (w - z) + f(w) * (z - x)) + -(f(x) * (w - z)) = f(w) * (z - x)
        f(z) * (w - x) - f(x) * (w - z) <= f(w) * (z - x)
        // f(w) (z - x) = f(w) (w - x) - f(w) (w - z)
        sub_sub_cancel(w, x, z)
        (w - x) - (w - z) = z - x
        mul_sub_distrib_right(f(w), w - x, w - z)
        f(w) * ((w - x) - (w - z)) = f(w) * (w - x) - f(w) * (w - z)
        f(w) * (z - x) = f(w) * (w - x) - f(w) * (w - z)
        f(z) * (w - x) - f(x) * (w - z) <= f(w) * (w - x) - f(w) * (w - z)
        // add f(w) (w - z) to both sides
        lte_add_right(f(z) * (w - x) - f(x) * (w - z), f(w) * (w - x) - f(w) * (w - z), f(w) * (w - z))
        (f(z) * (w - x) - f(x) * (w - z)) + f(w) * (w - z) <= (f(w) * (w - x) - f(w) * (w - z)) + f(w) * (w - z)
        add_assoc(f(w) * (w - x), -(f(w) * (w - z)), f(w) * (w - z))
        (f(w) * (w - x) + -(f(w) * (w - z))) + f(w) * (w - z) = f(w) * (w - x) + (-(f(w) * (w - z)) + f(w) * (w - z))
        add_comm(-(f(w) * (w - z)), f(w) * (w - z))
        -(f(w) * (w - z)) + f(w) * (w - z) = f(w) * (w - z) + -(f(w) * (w - z))
        add_neg_eq_zero(f(w) * (w - z))
        f(w) * (w - z) + -(f(w) * (w - z)) = Real.0
        f(w) * (w - x) + (-(f(w) * (w - z)) + f(w) * (w - z)) = f(w) * (w - x) + Real.0
        add_zero_right(f(w) * (w - x))
        f(w) * (w - x) + Real.0 = f(w) * (w - x)
        (f(w) * (w - x) - f(w) * (w - z)) + f(w) * (w - z) = f(w) * (w - x)
        (f(z) * (w - x) - f(x) * (w - z)) + f(w) * (w - z) <= f(w) * (w - x)
        // regroup the left side: f(z) (w - x) + (f(w) - f(x)) (w - z)
        add_assoc(f(z) * (w - x), -(f(x) * (w - z)), f(w) * (w - z))
        (f(z) * (w - x) + -(f(x) * (w - z))) + f(w) * (w - z) = f(z) * (w - x) + (-(f(x) * (w - z)) + f(w) * (w - z))
        add_comm(-(f(x) * (w - z)), f(w) * (w - z))
        -(f(x) * (w - z)) + f(w) * (w - z) = f(w) * (w - z) + -(f(x) * (w - z))
        f(z) * (w - x) + (-(f(x) * (w - z)) + f(w) * (w - z)) = f(z) * (w - x) + (f(w) * (w - z) + -(f(x) * (w - z)))
        f(w) * (w - z) - f(x) * (w - z) = f(w) * (w - z) + -(f(x) * (w - z))
        f(z) * (w - x) + (-(f(x) * (w - z)) + f(w) * (w - z)) = f(z) * (w - x) + (f(w) * (w - z) - f(x) * (w - z))
        mul_sub_distrib_right(f(w), f(x), w - z)
        (f(w) - f(x)) * (w - z) = f(w) * (w - z) - f(x) * (w - z)
        f(w) * (w - z) - f(x) * (w - z) = (f(w) - f(x)) * (w - z)
        f(z) * (w - x) + (f(w) * (w - z) - f(x) * (w - z)) = f(z) * (w - x) + (f(w) - f(x)) * (w - z)
        (f(z) * (w - x) + -(f(x) * (w - z))) + f(w) * (w - z) = f(z) * (w - x) + (f(w) * (w - z) + -(f(x) * (w - z)))
        (f(z) * (w - x) + -(f(x) * (w - z))) + f(w) * (w - z) = f(z) * (w - x) + (f(w) * (w - z) - f(x) * (w - z))
        (f(z) * (w - x) + -(f(x) * (w - z))) + f(w) * (w - z) = f(z) * (w - x) + (f(w) - f(x)) * (w - z)
        (f(z) * (w - x) - f(x) * (w - z)) + f(w) * (w - z) = (f(z) * (w - x) + -(f(x) * (w - z))) + f(w) * (w - z)
        (f(z) * (w - x) - f(x) * (w - z)) + f(w) * (w - z) = f(z) * (w - x) + (f(w) - f(x)) * (w - z)
        f(z) * (w - x) + (f(w) - f(x)) * (w - z) <= f(w) * (w - x)
        // subtract f(z) (w - x) from both sides
        lte_add_right(f(z) * (w - x) + (f(w) - f(x)) * (w - z), f(w) * (w - x), -(f(z) * (w - x)))
        (f(z) * (w - x) + (f(w) - f(x)) * (w - z)) + -(f(z) * (w - x)) <= f(w) * (w - x) + -(f(z) * (w - x))
        add_comm(f(z) * (w - x), (f(w) - f(x)) * (w - z))
        f(z) * (w - x) + (f(w) - f(x)) * (w - z) = (f(w) - f(x)) * (w - z) + f(z) * (w - x)
        sub_cancels((f(w) - f(x)) * (w - z), f(z) * (w - x))
        ((f(w) - f(x)) * (w - z) + f(z) * (w - x)) + -(f(z) * (w - x)) = (f(w) - f(x)) * (w - z)
        (f(z) * (w - x) + (f(w) - f(x)) * (w - z)) + -(f(z) * (w - x)) = (f(w) - f(x)) * (w - z)
        (f(w) - f(x)) * (w - z) <= f(w) * (w - x) - f(z) * (w - x)
        mul_sub_distrib_right(f(w), f(z), w - x)
        (f(w) - f(z)) * (w - x) = f(w) * (w - x) - f(z) * (w - x)
        (f(w) - f(x)) * (w - z) <= (f(w) - f(z)) * (w - x)
        // divide by (w - x), then by (w - z)
        inverse_of_positive_is_positive[Real](w - x)
        Real.0 < (w - x).inverse
        lt_imp_lte(Real.0, (w - x).inverse)
        Real.0 <= (w - x).inverse
        mul_le_mul_of_nonneg_right[Real]((f(w) - f(x)) * (w - z), (f(w) - f(z)) * (w - x), (w - x).inverse)
        ((f(w) - f(x)) * (w - z)) * (w - x).inverse <= ((f(w) - f(z)) * (w - x)) * (w - x).inverse
        mul_assoc(f(w) - f(z), w - x, (w - x).inverse)
        (f(w) - f(z)) * ((w - x) * (w - x).inverse) = ((f(w) - f(z)) * (w - x)) * (w - x).inverse
        (w - x) * (w - x).inverse = Real.1
        (f(w) - f(z)) * ((w - x) * (w - x).inverse) = (f(w) - f(z)) * Real.1
        mul_one_right(f(w) - f(z))
        (f(w) - f(z)) * Real.1 = f(w) - f(z)
        ((f(w) - f(z)) * (w - x)) * (w - x).inverse = f(w) - f(z)
        ((f(w) - f(x)) * (w - z)) * (w - x).inverse <= f(w) - f(z)
        lt_imp_minus_pos(z, w)
        (w - z).is_positive
        pos_gt_zero(w - z)
        w - z > Real.0
        inverse_of_positive_is_positive[Real](w - z)
        Real.0 < (w - z).inverse
        lt_imp_lte(Real.0, (w - z).inverse)
        Real.0 <= (w - z).inverse
        mul_le_mul_of_nonneg_right[Real](((f(w) - f(x)) * (w - z)) * (w - x).inverse, f(w) - f(z), (w - z).inverse)
        (((f(w) - f(x)) * (w - z)) * (w - x).inverse) * (w - z).inverse <= (f(w) - f(z)) * (w - z).inverse
        mul_assoc((f(w) - f(x)) * (w - z), (w - x).inverse, (w - z).inverse)
        ((f(w) - f(x)) * (w - z)) * ((w - x).inverse * (w - z).inverse) =
            (((f(w) - f(x)) * (w - z)) * (w - x).inverse) * (w - z).inverse
        real_mul_comm((w - x).inverse, (w - z).inverse)
        (w - x).inverse * (w - z).inverse = (w - z).inverse * (w - x).inverse
        ((f(w) - f(x)) * (w - z)) * ((w - x).inverse * (w - z).inverse) =
            ((f(w) - f(x)) * (w - z)) * ((w - z).inverse * (w - x).inverse)
        mul_assoc(f(w) - f(x), w - z, (w - z).inverse * (w - x).inverse)
        (f(w) - f(x)) * ((w - z) * ((w - z).inverse * (w - x).inverse)) =
            ((f(w) - f(x)) * (w - z)) * ((w - z).inverse * (w - x).inverse)
        mul_assoc(w - z, (w - z).inverse, (w - x).inverse)
        (w - z) * ((w - z).inverse * (w - x).inverse) = ((w - z) * (w - z).inverse) * (w - x).inverse
        lt_imp_ne_symm(Real.0, w - z)
        (w - z) != Real.0
        mul_inverse(w - z)
        (w - z) * (w - z).inverse = Real.1
        ((w - z) * (w - z).inverse) * (w - x).inverse = Real.1 * (w - x).inverse
        mul_one_left((w - x).inverse)
        Real.1 * (w - x).inverse = (w - x).inverse
        (w - z) * ((w - z).inverse * (w - x).inverse) = (w - x).inverse
        (f(w) - f(x)) * ((w - z) * ((w - z).inverse * (w - x).inverse)) = (f(w) - f(x)) * (w - x).inverse
        (((f(w) - f(x)) * (w - z)) * (w - x).inverse) * (w - z).inverse = (f(w) - f(x)) * (w - x).inverse
        (f(w) - f(x)) * (w - x).inverse <= (f(w) - f(z)) * (w - z).inverse
        (f(w) - f(x)) * (w - x).inverse = (f(w) - f(x)) / (w - x)
        (f(w) - f(z)) * (w - z).inverse = (f(w) - f(z)) / (w - z)
        (f(w) - f(x)) / (w - x) <= (f(w) - f(z)) / (w - z)
        secant_slope(f, x, w) = (f(w) - f(x)) / (w - x)
        secant_slope(f, z, w) = (f(w) - f(z)) / (w - z)
        secant_slope(f, x, w) <= secant_slope(f, z, w)
    }
}

/// A convex differentiable function has derivative at the left endpoint of a
/// chord bounded above by the chord slope.
theorem convex_imp_df_le_chord_slope(f: Real -> Real, df: Real -> Real, a: Real, b: Real, x: Real, y: Real) {
    convex_on(f, a, b) and is_derivative_fn(f, df) and a <= x and x < y and y <= b
    implies df(x) <= secant_slope(f, x, y)
} by {
    if convex_on(f, a, b) and is_derivative_fn(f, df) and a <= x and x < y and y <= b {
        if not df(x) <= secant_slope(f, x, y) {
            not_lte_imp_gt[Real](df(x), secant_slope(f, x, y))
            secant_slope(f, x, y) < df(x)
            lt_imp_minus_pos(secant_slope(f, x, y), df(x))
            (df(x) - secant_slope(f, x, y)).is_positive
            is_derivative_fn_at(f, df, x)
            has_derivative_at(f, x, df(x))
            has_derivative_at_delta(f, x, df(x), df(x) - secant_slope(f, x, y))
            let delta: Real satisfy {
                delta.is_positive and forall(z: Real) {
                    z != x and z.is_close(x, delta) implies
                    difference_quotient(f, x, z).is_close(df(x), df(x) - secant_slope(f, x, y))
                }
            }
            lt_imp_minus_pos(x, y)
            (y - x).is_positive
            eps_smaller_than_both(delta, y - x)
            let h: Real satisfy {
                h.is_positive and h < delta and h < y - x
            }
            let z = x + h
            lt_add_pos(x, h)
            x < z
            lt_imp_ne(x, z)
            z != x
            lt_add_right(h, y - x, x)
            h + x < (y - x) + x
            (y - x) + x = y
            h + x < y
            add_comm(h, x)
            h + x = x + h
            x + h < y
            z < y
            // z is close to x
            z - x = (x + h) - x
            add_comm(x, h)
            x + h = h + x
            (x + h) - x = (h + x) - x
            sub_cancels(h, x)
            h + x - x = h
            (x + h) - x = h
            z - x = h
            pos_imp_eq_abs(h)
            h = h.abs
            (z - x).abs = h.abs
            (z - x).abs = h
            h < delta
            (z - x).abs < delta
            z.is_close(x, delta)
            z != x and z.is_close(x, delta)
            forall(z0: Real) {
                z0 != x and z0.is_close(x, delta) implies difference_quotient(f, x, z0).is_close(df(x), df(x) - secant_slope(f, x, y))
            }
            difference_quotient(f, x, z).is_close(df(x), df(x) - secant_slope(f, x, y))
            close_imp_bounds(difference_quotient(f, x, z), df(x), df(x) - secant_slope(f, x, y))
            df(x) - (df(x) - secant_slope(f, x, y)) < difference_quotient(f, x, z)
            sub_minus_sub(df(x), secant_slope(f, x, y))
            df(x) - (df(x) - secant_slope(f, x, y)) = secant_slope(f, x, y)
            secant_slope(f, x, y) < difference_quotient(f, x, z)
            difference_quotient_eq_secant_slope(f, x, z)
            difference_quotient(f, x, z) = secant_slope(f, x, z)
            secant_slope(f, x, y) < secant_slope(f, x, z)
            convex_slope_second_arg_monotone(f, a, b, x, z, y)
            secant_slope(f, x, z) <= secant_slope(f, x, y)
            lt_of_lt_of_lte[Real](secant_slope(f, x, y), secant_slope(f, x, z), secant_slope(f, x, y))
            secant_slope(f, x, y) < secant_slope(f, x, y)
            not_lt_self(secant_slope(f, x, y))
            false
        }
        df(x) <= secant_slope(f, x, y)
    }
}

/// A convex differentiable function has derivative at the right endpoint of a
/// chord bounded below by the chord slope.
theorem convex_imp_chord_slope_le_df(f: Real -> Real, df: Real -> Real, a: Real, b: Real, x: Real, y: Real) {
    convex_on(f, a, b) and is_derivative_fn(f, df) and a <= x and x < y and y <= b
    implies secant_slope(f, x, y) <= df(y)
} by {
    if convex_on(f, a, b) and is_derivative_fn(f, df) and a <= x and x < y and y <= b {
        if not secant_slope(f, x, y) <= df(y) {
            not_lte_imp_gt[Real](secant_slope(f, x, y), df(y))
            df(y) < secant_slope(f, x, y)
            lt_imp_minus_pos(df(y), secant_slope(f, x, y))
            (secant_slope(f, x, y) - df(y)).is_positive
            is_derivative_fn_at(f, df, y)
            has_derivative_at(f, y, df(y))
            has_derivative_at_delta(f, y, df(y), secant_slope(f, x, y) - df(y))
            let delta: Real satisfy {
                delta.is_positive and forall(z: Real) {
                    z != y and z.is_close(y, delta) implies
                    difference_quotient(f, y, z).is_close(df(y), secant_slope(f, x, y) - df(y))
                }
            }
            lt_imp_minus_pos(x, y)
            (y - x).is_positive
            eps_smaller_than_both(delta, y - x)
            let h: Real satisfy {
                h.is_positive and h < delta and h < y - x
            }
            let z = y - h
            // x < z
            lt_add_right(h, y - x, x)
            h + x < (y - x) + x
            (y - x) + x = y
            h + x < y
            add_comm(h, x)
            h + x = x + h
            x + h < y
            (y - h) + h = y
            x + h < (y - h) + h
            lt_add_converse(x, y - h, h)
            x < z
            // z < y
            neg_pos_is_neg(h)
            (-h).is_negative
            neg_lt_zero(-h)
            -h < Real.0
            lt_add_right(-h, Real.0, y)
            -h + y < Real.0 + y
            add_comm(-h, y)
            -h + y = y + -h
            y - h = y + -h
            y + -h < y
            y - h < y
            z < y
            lt_imp_ne(z, y)
            z != y
            // z is close to y
            z - y = (y - h) - y
            (y - h) - y = -h
            z - y = -h
            abs_neg(h)
            (-h).abs = h.abs
            pos_imp_eq_abs(h)
            h = h.abs
            (z - y).abs = (-h).abs
            (z - y).abs = h
            h < delta
            (z - y).abs < delta
            z.is_close(y, delta)
            z != y and z.is_close(y, delta)
            forall(z0: Real) {
                z0 != y and z0.is_close(y, delta) implies difference_quotient(f, y, z0).is_close(df(y), secant_slope(f, x, y) - df(y))
            }
            difference_quotient(f, y, z).is_close(df(y), secant_slope(f, x, y) - df(y))
            close_imp_bounds(difference_quotient(f, y, z), df(y), secant_slope(f, x, y) - df(y))
            difference_quotient(f, y, z) < df(y) + (secant_slope(f, x, y) - df(y))
            secant_slope(f, x, y) - df(y) = secant_slope(f, x, y) + -df(y)
            df(y) + (secant_slope(f, x, y) - df(y)) = df(y) + (secant_slope(f, x, y) + -df(y))
            df(y) + (secant_slope(f, x, y) + -df(y)) = (df(y) + secant_slope(f, x, y)) + -df(y)
            add_comm(df(y), secant_slope(f, x, y))
            df(y) + secant_slope(f, x, y) = secant_slope(f, x, y) + df(y)
            (df(y) + secant_slope(f, x, y)) + -df(y) = (secant_slope(f, x, y) + df(y)) + -df(y)
            sub_cancels(secant_slope(f, x, y), df(y))
            (secant_slope(f, x, y) + df(y)) + -df(y) = secant_slope(f, x, y)
            df(y) + (secant_slope(f, x, y) - df(y)) = secant_slope(f, x, y)
            difference_quotient(f, y, z) < secant_slope(f, x, y)
            difference_quotient_swap_eq_secant_slope(f, y, z)
            difference_quotient(f, y, z) = secant_slope(f, z, y)
            secant_slope(f, z, y) < secant_slope(f, x, y)
            convex_slope_first_arg_monotone(f, a, b, x, z, y)
            secant_slope(f, x, y) <= secant_slope(f, z, y)
            lte_lt_trans(secant_slope(f, x, y), secant_slope(f, z, y), secant_slope(f, x, y))
            secant_slope(f, x, y) < secant_slope(f, x, y)
            not_lt_self(secant_slope(f, x, y))
            false
        }
        secant_slope(f, x, y) <= df(y)
    }
}

/// A convex differentiable function has nondecreasing derivative on the
/// interior of the interval.
theorem convex_imp_derivative_nondecreasing(f: Real -> Real, df: Real -> Real, a: Real, b: Real, x: Real, y: Real) {
    convex_on(f, a, b) and is_derivative_fn(f, df) and a < x and x < y and y < b
    implies df(x) <= df(y)
} by {
    if convex_on(f, a, b) and is_derivative_fn(f, df) and a < x and x < y and y < b {
        lt_imp_lte(a, x)
        a <= x
        lt_imp_lte(y, b)
        y <= b
        convex_imp_df_le_chord_slope(f, df, a, b, x, y)
        df(x) <= secant_slope(f, x, y)
        convex_imp_chord_slope_le_df(f, df, a, b, x, y)
        secant_slope(f, x, y) <= df(y)
        lte_trans(df(x), secant_slope(f, x, y), df(y))
        df(x) <= df(y)
    }
}

/// A convex differentiable function has nondecreasing derivative throughout
/// the interior of the interval.
theorem convex_imp_derivative_fn_nondecreasing_on(f: Real -> Real, df: Real -> Real, a: Real, b: Real) {
    convex_on(f, a, b) and is_derivative_fn(f, df) and a < b
    implies forall(x: Real, y: Real) { a < x and x < y and y < b implies df(x) <= df(y) }
} by {
    if convex_on(f, a, b) and is_derivative_fn(f, df) and a < b {
        forall(x: Real, y: Real) {
            if a < x and x < y and y < b {
                convex_imp_derivative_nondecreasing(f, df, a, b, x, y)
                df(x) <= df(y)
            }
        }
    }
}

/// A chord-slope inequality at an interior point implies the convexity
/// inequality: for x < z < y with z = (1 - t) x + t y,
/// slope(x, z) <= slope(z, y) gives f(z) <= (1 - t) f(x) + t f(y).
theorem slope_inequality_imp_convex_combination(f: Real -> Real, x: Real, z: Real, y: Real, t: Real) {
    x < z and z < y and z = (Real.1 - t) * x + t * y and
    secant_slope(f, x, z) <= secant_slope(f, z, y)
    implies f(z) <= (Real.1 - t) * f(x) + t * f(y)
} by {
    if x < z and z < y and z = (Real.1 - t) * x + t * y and
       secant_slope(f, x, z) <= secant_slope(f, z, y) {
        secant_slope(f, x, z) = (f(z) - f(x)) / (z - x)
        secant_slope(f, z, y) = (f(y) - f(z)) / (y - z)
        (f(z) - f(x)) / (z - x) <= (f(y) - f(z)) / (y - z)
        // positive denominators
        lt_imp_minus_pos(x, z)
        (z - x).is_positive
        lt_imp_minus_pos(z, y)
        (y - z).is_positive
        pos_gt_zero(z - x)
        z - x > Real.0
        pos_gt_zero(y - z)
        y - z > Real.0
        lt_trans[Real](x, z, y)
        x < y
        lt_imp_minus_pos(x, y)
        (y - x).is_positive
        pos_gt_zero(y - x)
        y - x > Real.0
        // cross-multiply by (z - x) and (y - z)
        lt_imp_lte(Real.0, z - x)
        Real.0 <= z - x
        mul_le_mul_of_nonneg_right[Real]((f(z) - f(x)) / (z - x), (f(y) - f(z)) / (y - z), z - x)
        ((f(z) - f(x)) / (z - x)) * (z - x) <= ((f(y) - f(z)) / (y - z)) * (z - x)
        lt_imp_ne_symm(Real.0, z - x)
        (z - x) != Real.0
        div_mul_cancel_denominator(f(z) - f(x), z - x)
        ((f(z) - f(x)) / (z - x)) * (z - x) = f(z) - f(x)
        f(z) - f(x) <= ((f(y) - f(z)) / (y - z)) * (z - x)
        lt_imp_lte(Real.0, y - z)
        Real.0 <= y - z
        mul_le_mul_of_nonneg_right[Real](f(z) - f(x), ((f(y) - f(z)) / (y - z)) * (z - x), y - z)
        (f(z) - f(x)) * (y - z) <= (((f(y) - f(z)) / (y - z)) * (z - x)) * (y - z)
        mul_assoc((f(y) - f(z)) / (y - z), z - x, y - z)
        ((f(y) - f(z)) / (y - z)) * ((z - x) * (y - z)) = (((f(y) - f(z)) / (y - z)) * (z - x)) * (y - z)
        real_mul_comm(z - x, y - z)
        (z - x) * (y - z) = (y - z) * (z - x)
        ((f(y) - f(z)) / (y - z)) * ((z - x) * (y - z)) = ((f(y) - f(z)) / (y - z)) * ((y - z) * (z - x))
        mul_assoc((f(y) - f(z)) / (y - z), y - z, z - x)
        ((f(y) - f(z)) / (y - z)) * ((y - z) * (z - x)) = (((f(y) - f(z)) / (y - z)) * (y - z)) * (z - x)
        lt_imp_ne_symm(Real.0, y - z)
        (y - z) != Real.0
        div_mul_cancel_denominator(f(y) - f(z), y - z)
        ((f(y) - f(z)) / (y - z)) * (y - z) = f(y) - f(z)
        (((f(y) - f(z)) / (y - z)) * (y - z)) * (z - x) = (f(y) - f(z)) * (z - x)
        (((f(y) - f(z)) / (y - z)) * (z - x)) * (y - z) = (f(y) - f(z)) * (z - x)
        (f(z) - f(x)) * (y - z) <= (f(y) - f(z)) * (z - x)
        // expand both sides
        mul_sub_distrib_right(f(z), f(x), y - z)
        (f(z) - f(x)) * (y - z) = f(z) * (y - z) - f(x) * (y - z)
        mul_sub_distrib_right(f(y), f(z), z - x)
        (f(y) - f(z)) * (z - x) = f(y) * (z - x) - f(z) * (z - x)
        f(z) * (y - z) - f(x) * (y - z) <= f(y) * (z - x) - f(z) * (z - x)
        // add f(z)(z - x) to both sides
        lte_add_right(f(z) * (y - z) - f(x) * (y - z), f(y) * (z - x) - f(z) * (z - x), f(z) * (z - x))
        (f(z) * (y - z) - f(x) * (y - z)) + f(z) * (z - x) <= (f(y) * (z - x) - f(z) * (z - x)) + f(z) * (z - x)
        (f(y) * (z - x) - f(z) * (z - x)) + f(z) * (z - x) = f(y) * (z - x)
        (f(z) * (y - z) - f(x) * (y - z)) + f(z) * (z - x) <= f(y) * (z - x)
        // regroup: f(z)((y - z) + (z - x)) - f(x)(y - z)
        (y - z) + (z - x) = y - x
        add_assoc(f(z) * (y - z), -(f(x) * (y - z)), f(z) * (z - x))
        (f(z) * (y - z) + -(f(x) * (y - z))) + f(z) * (z - x) = f(z) * (y - z) + (-(f(x) * (y - z)) + f(z) * (z - x))
        add_comm(-(f(x) * (y - z)), f(z) * (z - x))
        -(f(x) * (y - z)) + f(z) * (z - x) = f(z) * (z - x) + -(f(x) * (y - z))
        f(z) * (y - z) + (-(f(x) * (y - z)) + f(z) * (z - x)) = f(z) * (y - z) + (f(z) * (z - x) - f(x) * (y - z))
        add_assoc(f(z) * (y - z), f(z) * (z - x), -(f(x) * (y - z)))
        (f(z) * (y - z) + f(z) * (z - x)) + -(f(x) * (y - z)) = f(z) * (y - z) + (f(z) * (z - x) + -(f(x) * (y - z)))
        mul_distrib_right(f(z), y - z, z - x)
        f(z) * ((y - z) + (z - x)) = f(z) * (y - z) + f(z) * (z - x)
        f(z) * ((y - z) + (z - x)) = f(z) * (y - x)
        (f(z) * (y - z) + f(z) * (z - x)) + -(f(x) * (y - z)) = f(z) * (y - x) + -(f(x) * (y - z))
        f(z) * (y - x) - f(x) * (y - z) = f(z) * (y - x) + -(f(x) * (y - z))
        (f(z) * (y - z) - f(x) * (y - z)) + f(z) * (z - x) = f(z) * (y - x) - f(x) * (y - z)
        f(z) * (y - x) - f(x) * (y - z) <= f(y) * (z - x)
        // add f(x)(y - z) to both sides
        lte_add_right(f(z) * (y - x) - f(x) * (y - z), f(y) * (z - x), f(x) * (y - z))
        (f(z) * (y - x) - f(x) * (y - z)) + f(x) * (y - z) <= f(y) * (z - x) + f(x) * (y - z)
        (f(z) * (y - x) - f(x) * (y - z)) + f(x) * (y - z) = f(z) * (y - x)
        f(z) * (y - x) <= f(y) * (z - x) + f(x) * (y - z)
        // z - x = t (y - x) and y - z = (1 - t)(y - x)
        z = (Real.1 - t) * x + t * y
        z - x = (Real.1 - t) * x + t * y - x
        mul_sub_distrib_left(Real.1, t, x)
        (Real.1 - t) * x = Real.1 * x - t * x
        mul_one_left(x)
        Real.1 * x = x
        (Real.1 - t) * x = x - t * x
        add_assoc(x, -(t * x), -x)
        (x + -(t * x)) + -x = x + (-(t * x) + -x)
        add_comm(-(t * x), -x)
        -(t * x) + -x = -x + -(t * x)
        add_assoc(x, -x, -(t * x))
        (x + -x) + -(t * x) = x + (-x + -(t * x))
        add_neg_eq_zero(x)
        x + -x = Real.0
        (x + -x) + -(t * x) = Real.0 + -(t * x)
        add_zero_left(-(t * x))
        Real.0 + -(t * x) = -(t * x)
        x + (-x + -(t * x)) = -(t * x)
        x + (-(t * x) + -x) = -(t * x)
        (x + -(t * x)) + -x = -(t * x)
        (x - t * x) + -x = -(t * x)
        add_assoc(x + -(t * x), t * y, -x)
        ((x + -(t * x)) + t * y) + -x = (x + -(t * x)) + (t * y + -x)
        add_comm(t * y, -x)
        t * y + -x = -x + t * y
        (x + -(t * x)) + (t * y + -x) = (x + -(t * x)) + (-x + t * y)
        add_assoc(x + -(t * x), -x, t * y)
        ((x + -(t * x)) + -x) + t * y = (x + -(t * x)) + (-x + t * y)
        ((x + -(t * x)) + -x) + t * y = -(t * x) + t * y
        ((x + -(t * x)) + t * y) + -x = -(t * x) + t * y
        (x - t * x) + t * y + -x = -(t * x) + t * y
        add_comm(-(t * x), t * y)
        -(t * x) + t * y = t * y + -(t * x)
        (x - t * x) + t * y + -x = t * y - t * x
        mul_sub_distrib_right(t, y, x)
        t * (y - x) = t * y - t * x
        (Real.1 - t) * x + t * y - x = t * (y - x)
        z - x = t * (y - x)
        // y - z = (1 - t)(y - x)
        y - z = y - ((Real.1 - t) * x + t * y)
        neg_distrib((Real.1 - t) * x, t * y)
        -((Real.1 - t) * x + t * y) = -((Real.1 - t) * x) + -(t * y)
        add_assoc(y, -((Real.1 - t) * x), -(t * y))
        (y + -((Real.1 - t) * x)) + -(t * y) = y + (-((Real.1 - t) * x) + -(t * y))
        add_comm(-((Real.1 - t) * x), -(t * y))
        -((Real.1 - t) * x) + -(t * y) = -(t * y) + -((Real.1 - t) * x)
        y + (-((Real.1 - t) * x) + -(t * y)) = y + (-(t * y) + -((Real.1 - t) * x))
        add_assoc(y, -(t * y), -((Real.1 - t) * x))
        (y + -(t * y)) + -((Real.1 - t) * x) = y + (-(t * y) + -((Real.1 - t) * x))
        mul_sub_distrib_right(Real.1, t, y)
        (Real.1 - t) * y = Real.1 * y - t * y
        mul_one_left(y)
        Real.1 * y = y
        (Real.1 - t) * y = y - t * y
        y - t * y = y + -(t * y)
        y + (-(t * y) + -((Real.1 - t) * x)) = (y - t * y) + -((Real.1 - t) * x)
        mul_sub_distrib_right(Real.1 - t, y, x)
        (Real.1 - t) * (y - x) = (Real.1 - t) * y - (Real.1 - t) * x
        (Real.1 - t) * y - (Real.1 - t) * x = (Real.1 - t) * y + -((Real.1 - t) * x)
        y - ((Real.1 - t) * x + t * y) = (Real.1 - t) * (y - x)
        y - z = (Real.1 - t) * (y - x)
        // rewrite the right side of the inequality
        f(y) * (z - x) = f(y) * (t * (y - x))
        real_mul_comm(t, f(y))
        t * f(y) = f(y) * t
        f(y) * (t * (y - x)) = (t * f(y)) * (y - x)
        f(x) * (y - z) = f(x) * ((Real.1 - t) * (y - x))
        real_mul_comm(Real.1 - t, f(x))
        (Real.1 - t) * f(x) = f(x) * (Real.1 - t)
        f(x) * ((Real.1 - t) * (y - x)) = ((Real.1 - t) * f(x)) * (y - x)
        f(y) * (z - x) + f(x) * (y - z) = (t * f(y)) * (y - x) + ((Real.1 - t) * f(x)) * (y - x)
        mul_distrib_left(t * f(y), (Real.1 - t) * f(x), y - x)
        (t * f(y) + (Real.1 - t) * f(x)) * (y - x) = (t * f(y)) * (y - x) + ((Real.1 - t) * f(x)) * (y - x)
        f(y) * (z - x) + f(x) * (y - z) = (t * f(y) + (Real.1 - t) * f(x)) * (y - x)
        f(z) * (y - x) <= (t * f(y) + (Real.1 - t) * f(x)) * (y - x)
        // divide by (y - x)
        inverse_of_positive_is_positive[Real](y - x)
        Real.0 < (y - x).inverse
        lt_imp_lte(Real.0, (y - x).inverse)
        Real.0 <= (y - x).inverse
        mul_right_scalar_map(y - x, f(z)) = f(z) * (y - x)
        mul_right_scalar_map(y - x, t * f(y) + (Real.1 - t) * f(x)) = (t * f(y) + (Real.1 - t) * f(x)) * (y - x)
        mul_right_scalar_map(y - x, f(z)) <= mul_right_scalar_map(y - x, t * f(y) + (Real.1 - t) * f(x))
        mul_right_scalar_map_reflects_le_of_positive[Real](y - x, f(z), t * f(y) + (Real.1 - t) * f(x))
        f(z) <= t * f(y) + (Real.1 - t) * f(x)
        add_comm(t * f(y), (Real.1 - t) * f(x))
        t * f(y) + (Real.1 - t) * f(x) = (Real.1 - t) * f(x) + t * f(y)
        f(z) <= (Real.1 - t) * f(x) + t * f(y)
    }
}

/// A strict convex combination of x and y with weight t in (0, 1) lies
/// strictly between x and y.
theorem convex_combination_between(x: Real, y: Real, z: Real, t: Real) {
    x < y and Real.0 < t and t < Real.1 and z = (Real.1 - t) * x + t * y
    implies x < z and z < y
} by {
    if x < y and Real.0 < t and t < Real.1 and z = (Real.1 - t) * x + t * y {
        z = (Real.1 - t) * x + t * y
        z - x = (Real.1 - t) * x + t * y - x
        mul_sub_distrib_left(Real.1, t, x)
        (Real.1 - t) * x = Real.1 * x - t * x
        mul_one_left(x)
        Real.1 * x = x
        (Real.1 - t) * x = x - t * x
        add_assoc(x, -(t * x), -x)
        (x + -(t * x)) + -x = x + (-(t * x) + -x)
        add_comm(-(t * x), -x)
        -(t * x) + -x = -x + -(t * x)
        add_assoc(x, -x, -(t * x))
        (x + -x) + -(t * x) = x + (-x + -(t * x))
        add_neg_eq_zero(x)
        x + -x = Real.0
        (x + -x) + -(t * x) = Real.0 + -(t * x)
        add_zero_left(-(t * x))
        Real.0 + -(t * x) = -(t * x)
        x + (-x + -(t * x)) = -(t * x)
        x + (-(t * x) + -x) = -(t * x)
        (x + -(t * x)) + -x = -(t * x)
        (x - t * x) + -x = -(t * x)
        add_assoc(x + -(t * x), t * y, -x)
        ((x + -(t * x)) + t * y) + -x = (x + -(t * x)) + (t * y + -x)
        add_comm(t * y, -x)
        t * y + -x = -x + t * y
        (x + -(t * x)) + (t * y + -x) = (x + -(t * x)) + (-x + t * y)
        add_assoc(x + -(t * x), -x, t * y)
        ((x + -(t * x)) + -x) + t * y = (x + -(t * x)) + (-x + t * y)
        ((x + -(t * x)) + -x) + t * y = -(t * x) + t * y
        ((x + -(t * x)) + t * y) + -x = -(t * x) + t * y
        (x - t * x) + t * y + -x = -(t * x) + t * y
        add_comm(-(t * x), t * y)
        -(t * x) + t * y = t * y + -(t * x)
        (x - t * x) + t * y + -x = t * y - t * x
        mul_sub_distrib_right(t, y, x)
        t * (y - x) = t * y - t * x
        (Real.1 - t) * x + t * y - x = t * (y - x)
        z - x = t * (y - x)
        // z - x > 0
        lt_imp_minus_pos(x, y)
        (y - x).is_positive
        pos_gt_zero(y - x)
        y - x > Real.0
        mul_pos_pos(t, y - x)
        Real.0 < t * (y - x)
        Real.0 < z - x
        gt_zero_imp_pos(z - x)
        (z - x).is_positive
        diff_pos_imp_lt(x, z)
        x < z
        // y - z = (1 - t)(y - x)
        y - z = y - ((Real.1 - t) * x + t * y)
        neg_distrib((Real.1 - t) * x, t * y)
        -((Real.1 - t) * x + t * y) = -((Real.1 - t) * x) + -(t * y)
        add_assoc(y, -((Real.1 - t) * x), -(t * y))
        (y + -((Real.1 - t) * x)) + -(t * y) = y + (-((Real.1 - t) * x) + -(t * y))
        add_comm(-((Real.1 - t) * x), -(t * y))
        -((Real.1 - t) * x) + -(t * y) = -(t * y) + -((Real.1 - t) * x)
        y + (-((Real.1 - t) * x) + -(t * y)) = y + (-(t * y) + -((Real.1 - t) * x))
        add_assoc(y, -(t * y), -((Real.1 - t) * x))
        (y + -(t * y)) + -((Real.1 - t) * x) = y + (-(t * y) + -((Real.1 - t) * x))
        mul_sub_distrib_right(Real.1, t, y)
        (Real.1 - t) * y = Real.1 * y - t * y
        mul_one_left(y)
        Real.1 * y = y
        (Real.1 - t) * y = y - t * y
        y - t * y = y + -(t * y)
        y + (-(t * y) + -((Real.1 - t) * x)) = (y - t * y) + -((Real.1 - t) * x)
        mul_sub_distrib_right(Real.1 - t, y, x)
        (Real.1 - t) * (y - x) = (Real.1 - t) * y - (Real.1 - t) * x
        (Real.1 - t) * y - (Real.1 - t) * x = (Real.1 - t) * y + -((Real.1 - t) * x)
        y - ((Real.1 - t) * x + t * y) = (Real.1 - t) * (y - x)
        y - z = (Real.1 - t) * (y - x)
        // y - z > 0
        lt_imp_minus_pos(t, Real.1)
        (Real.1 - t).is_positive
        pos_gt_zero(Real.1 - t)
        Real.1 - t > Real.0
        Real.0 < Real.1 - t
        Real.0 < y - x
        Real.0 < Real.1 - t and Real.0 < y - x
        mul_pos_pos(Real.1 - t, y - x)
        Real.0 < (Real.1 - t) * (y - x)
        Real.0 < y - z
        gt_zero_imp_pos(y - z)
        (y - z).is_positive
        diff_pos_imp_lt(z, y)
        z < y
        x < z and z < y
    }
}

/// The chord condition of a differentiable function with nondecreasing
/// derivative: the value at each convex combination lies below the chord.
theorem derivative_nondecreasing_imp_convex_condition(f: Real -> Real, df: Real -> Real, a: Real, b: Real) {
    a < b and is_derivative_fn(f, df) and
    (forall(u: Real, v: Real) { u <= v implies df(u) <= df(v) })
    implies forall(x: Real, y: Real, t: Real) {
        a <= x and x <= y and y <= b and Real.0 <= t and t <= Real.1
        implies f((Real.1 - t) * x + t * y) <= (Real.1 - t) * f(x) + t * f(y)
    }
} by {
    if a < b and is_derivative_fn(f, df) and
       (forall(u: Real, v: Real) { u <= v implies df(u) <= df(v) }) {
        is_derivative_fn_imp_continuous(f, df)
        continuous(f)
        forall(x: Real, y: Real, t: Real) {
            if a <= x and x <= y and y <= b and Real.0 <= t and t <= Real.1 {
                let z = (Real.1 - t) * x + t * y
                if x < y {
                    if t = Real.0 {
                        z = (Real.1 - t) * x + t * y
                        (Real.1 - Real.0) * x + Real.0 * y = x
                        z = x
                        f(z) = f(x)
                        (Real.1 - Real.0) * f(x) + Real.0 * f(y) = f(x)
                        (Real.1 - t) * f(x) + t * f(y) = (Real.1 - Real.0) * f(x) + Real.0 * f(y)
                        (Real.1 - t) * f(x) + t * f(y) = f(x)
                        lte_refl(f(x))
                        f(x) <= f(x)
                        f(z) <= (Real.1 - t) * f(x) + t * f(y)
                        f(z) = f((Real.1 - t) * x + t * y)
                        f((Real.1 - t) * x + t * y) <= (Real.1 - t) * f(x) + t * f(y)
                    } else {
                        if t = Real.1 {
                            z = (Real.1 - t) * x + t * y
                            add_neg_eq_zero(Real.1)
                            Real.1 + -Real.1 = Real.0
                            Real.1 - Real.1 = Real.1 + -Real.1
                            Real.1 - Real.1 = Real.0
                            (Real.1 - Real.1) * x = Real.0 * x
                            mul_zero_left(x)
                            Real.0 * x = Real.0
                            mul_one_left(y)
                            Real.1 * y = y
                            Real.0 + Real.1 * y = Real.0 + y
                            add_zero_left(y)
                            Real.0 + y = y
                            (Real.1 - Real.1) * x + Real.1 * y = y
                            z = y
                            f(z) = f(y)
                            (Real.1 - Real.1) * f(x) = Real.0 * f(x)
                            mul_zero_left(f(x))
                            Real.0 * f(x) = Real.0
                            mul_one_left(f(y))
                            Real.1 * f(y) = f(y)
                            Real.0 + Real.1 * f(y) = Real.0 + f(y)
                            add_zero_left(f(y))
                            Real.0 + f(y) = f(y)
                            (Real.1 - Real.1) * f(x) + Real.1 * f(y) = f(y)
                            (Real.1 - t) * f(x) + t * f(y) = (Real.1 - Real.1) * f(x) + Real.1 * f(y)
                            (Real.1 - t) * f(x) + t * f(y) = f(y)
                            lte_refl(f(y))
                            f(y) <= f(y)
                            f(z) <= (Real.1 - t) * f(x) + t * f(y)
                            f(z) = f((Real.1 - t) * x + t * y)
                            f((Real.1 - t) * x + t * y) <= (Real.1 - t) * f(x) + t * f(y)
                        } else {
                            // 0 < t and t < 1
                            not t = Real.0
                            t != Real.0
                            Real.0 != t
                            Real.0 <= t and Real.0 != t
                            lt_of_lte_of_ne(Real.0, t)
                            Real.0 < t
                            not t = Real.1
                            t != Real.1
                            t <= Real.1 and t != Real.1
                            lt_of_lte_of_ne(t, Real.1)
                            t < Real.1
                            // x < z < y
                            convex_combination_between(x, y, z, t)
                            x < z and z < y
                            x < z
                            z < y
                            // mean value theorem on (x, z)
                            mean_value_theorem(f, df, x, z)
                            let c1: Real satisfy {
                                x < c1 and c1 < z and has_derivative_at(f, c1, secant_slope(f, x, z))
                            }
                            x < c1 and c1 < z
                            has_derivative_at(f, c1, secant_slope(f, x, z))
                            is_derivative_fn_at(f, df, c1)
                            has_derivative_at(f, c1, df(c1))
                            has_derivative_at_unique(f, c1, secant_slope(f, x, z), df(c1))
                            secant_slope(f, x, z) = df(c1)
                            // mean value theorem on (z, y)
                            mean_value_theorem(f, df, z, y)
                            let c2: Real satisfy {
                                z < c2 and c2 < y and has_derivative_at(f, c2, secant_slope(f, z, y))
                            }
                            z < c2 and c2 < y
                            has_derivative_at(f, c2, secant_slope(f, z, y))
                            is_derivative_fn_at(f, df, c2)
                            has_derivative_at(f, c2, df(c2))
                            has_derivative_at_unique(f, c2, secant_slope(f, z, y), df(c2))
                            secant_slope(f, z, y) = df(c2)
                            // c1 < c2 and df(c1) <= df(c2)
                            lt_trans[Real](c1, z, c2)
                            c1 < c2
                            lt_imp_lte(c1, c2)
                            c1 <= c2
                            forall(u: Real, v: Real) { u <= v implies df(u) <= df(v) }
                            df(c1) <= df(c2)
                            secant_slope(f, x, z) <= secant_slope(f, z, y)
                            slope_inequality_imp_convex_combination(f, x, z, y, t)
                            f(z) <= (Real.1 - t) * f(x) + t * f(y)
                            z = (Real.1 - t) * x + t * y
                            f(z) = f((Real.1 - t) * x + t * y)
                            f((Real.1 - t) * x + t * y) <= (Real.1 - t) * f(x) + t * f(y)
                        }
                    }
                } else {
                    // x = y
                    not x < y
                    not_lt_imp_gte[Real](x, y)
                    y <= x
                    lte_antisymm[Real](x, y)
                    x = y
                    (Real.1 - t) * x + t * y = (Real.1 - t) * x + t * x
                    mul_distrib_left(Real.1 - t, t, x)
                    (Real.1 - t + t) * x = (Real.1 - t) * x + t * x
                    add_assoc(Real.1, -t, t)
                    (Real.1 + -t) + t = Real.1 + (-t + t)
                    add_neg_eq_zero(t)
                    -t + t = Real.0
                    Real.1 + (-t + t) = Real.1 + Real.0
                    add_zero_right(Real.1)
                    Real.1 + Real.0 = Real.1
                    (Real.1 + -t) + t = Real.1
                    Real.1 - t + t = Real.1
                    (Real.1 - t) * x + t * x = Real.1 * x
                    mul_one_left(x)
                    Real.1 * x = x
                    (Real.1 - t) * x + t * x = x
                    z = x
                    f(z) = f(x)
                    (Real.1 - t) * f(x) + t * f(y) = (Real.1 - t) * f(x) + t * f(x)
                    mul_distrib_left(Real.1 - t, t, f(x))
                    (Real.1 - t + t) * f(x) = (Real.1 - t) * f(x) + t * f(x)
                    (Real.1 + -t) + t = Real.1 + (-t + t)
                    Real.1 + (-t + t) = Real.1 + Real.0
                    Real.1 + Real.0 = Real.1
                    (Real.1 + -t) + t = Real.1
                    Real.1 - t + t = Real.1
                    (Real.1 - t) * f(x) + t * f(x) = Real.1 * f(x)
                    mul_one_left(f(x))
                    Real.1 * f(x) = f(x)
                    (Real.1 - t) * f(x) + t * f(x) = f(x)
                    (Real.1 - t) * f(x) + t * f(y) = f(x)
                    lte_refl(f(x))
                    f(x) <= f(x)
                    f(z) <= (Real.1 - t) * f(x) + t * f(y)
                    z = (Real.1 - t) * x + t * y
                    f(z) = f((Real.1 - t) * x + t * y)
                    f((Real.1 - t) * x + t * y) <= (Real.1 - t) * f(x) + t * f(y)
                }
            }
        }
        forall(x: Real, y: Real, t: Real) { a <= x and x <= y and y <= b and Real.0 <= t and t <= Real.1 implies f((Real.1 - t) * x + t * y) <= (Real.1 - t) * f(x) + t * f(y) }
    }
}

/// A differentiable function with nondecreasing derivative is convex on the
/// interval.
theorem derivative_nondecreasing_imp_convex(f: Real -> Real, df: Real -> Real, a: Real, b: Real) {
    a < b and is_derivative_fn(f, df) and
    (forall(u: Real, v: Real) { u <= v implies df(u) <= df(v) })
    implies convex_on(f, a, b)
} by {
    if a < b and is_derivative_fn(f, df) and
       (forall(u: Real, v: Real) { u <= v implies df(u) <= df(v) }) {
        derivative_nondecreasing_imp_convex_condition(f, df, a, b)
        forall(x: Real, y: Real, t: Real) { a <= x and x <= y and y <= b and Real.0 <= t and t <= Real.1 implies f((Real.1 - t) * x + t * y) <= (Real.1 - t) * f(x) + t * f(y) }
        convex_on(f, a, b) = forall(x: Real, y: Real, t: Real) {
            a <= x and x <= y and y <= b and Real.0 <= t and t <= Real.1
            implies f((Real.1 - t) * x + t * y) <= (Real.1 - t) * f(x) + t * f(y)
        }
        convex_on(f, a, b)
    }
}

/// A twice differentiable convex function has nonnegative second derivative on
/// the interior of the interval: since the first derivative is nondecreasing,
/// every difference quotient of df is nonnegative, and so is its limit.
theorem convex_imp_second_derivative_nonneg(f: Real -> Real, df: Real -> Real, ddf: Real -> Real, a: Real, b: Real, x0: Real) {
    convex_on(f, a, b) and is_derivative_fn(f, df) and is_derivative_fn(df, ddf) and
    a < x0 and x0 < b
    implies Real.0 <= ddf(x0)
} by {
    if convex_on(f, a, b) and is_derivative_fn(f, df) and is_derivative_fn(df, ddf) and
       a < x0 and x0 < b {
        if ddf(x0) < Real.0 {
            // eps = -ddf(x0) is positive
            lt_add_right(ddf(x0), Real.0, -ddf(x0))
            ddf(x0) + -ddf(x0) < Real.0 + -ddf(x0)
            add_neg_eq_zero(ddf(x0))
            ddf(x0) + -ddf(x0) = Real.0
            add_zero_left(-ddf(x0))
            Real.0 + -ddf(x0) = -ddf(x0)
            Real.0 < -ddf(x0)
            gt_zero_imp_pos(-ddf(x0))
            (-ddf(x0)).is_positive
            is_derivative_fn_at(df, ddf, x0)
            has_derivative_at(df, x0, ddf(x0))
            has_derivative_at_delta(df, x0, ddf(x0), -ddf(x0))
            let delta: Real satisfy {
                delta.is_positive and forall(z: Real) {
                    z != x0 and z.is_close(x0, delta) implies difference_quotient(df, x0, z).is_close(ddf(x0), -ddf(x0))
                }
            }
            lt_imp_minus_pos(x0, b)
            (b - x0).is_positive
            eps_smaller_than_both(delta, b - x0)
            let h: Real satisfy {
                h.is_positive and h < delta and h < b - x0
            }
            let z = x0 + h
            lt_add_pos(x0, h)
            x0 < z
            lt_imp_ne(x0, z)
            z != x0
            lt_add_right(h, b - x0, x0)
            h + x0 < (b - x0) + x0
            (b - x0) + x0 = b
            h + x0 < b
            add_comm(h, x0)
            h + x0 = x0 + h
            x0 + h < b
            z < b
            // z is close to x0
            z - x0 = (x0 + h) - x0
            add_comm(x0, h)
            x0 + h = h + x0
            (x0 + h) - x0 = (h + x0) - x0
            sub_cancels(h, x0)
            h + x0 - x0 = h
            (x0 + h) - x0 = h
            z - x0 = h
            pos_imp_eq_abs(h)
            h = h.abs
            (z - x0).abs = h.abs
            (z - x0).abs = h
            h < delta
            (z - x0).abs < delta
            z.is_close(x0, delta)
            z != x0 and z.is_close(x0, delta)
            forall(z0: Real) {
                z0 != x0 and z0.is_close(x0, delta) implies difference_quotient(df, x0, z0).is_close(ddf(x0), -ddf(x0))
            }
            difference_quotient(df, x0, z).is_close(ddf(x0), -ddf(x0))
            close_imp_bounds(difference_quotient(df, x0, z), ddf(x0), -ddf(x0))
            difference_quotient(df, x0, z) < ddf(x0) + -ddf(x0)
            ddf(x0) + -ddf(x0) = Real.0
            difference_quotient(df, x0, z) < Real.0
            // the difference quotient is nonnegative
            convex_imp_derivative_nondecreasing(f, df, a, b, x0, z)
            df(x0) <= df(z)
            lte_imp_zero_le_sub(df(x0), df(z))
            Real.0 <= df(z) - df(x0)
            lt_imp_minus_pos(x0, z)
            (z - x0).is_positive
            ge_zero_div_pos_ge_zero(df(z) - df(x0), z - x0)
            Real.0 <= (df(z) - df(x0)) / (z - x0)
            difference_quotient(df, x0, z) = (df(z) - df(x0)) / (z - x0)
            Real.0 <= difference_quotient(df, x0, z)
            lt_of_lte_of_lt[Real](Real.0, difference_quotient(df, x0, z), Real.0)
            Real.0 < Real.0
            not_lt_self(Real.0)
            false
        }
        not ddf(x0) < Real.0
        not_lt_imp_gte[Real](ddf(x0), Real.0)
        ddf(x0) >= Real.0
        Real.0 <= ddf(x0)
    }
}
