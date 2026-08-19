/// Lipschitz continuity and its consequences.
///
/// This file defines Lipschitz functions and proves that a Lipschitz function
/// is uniformly continuous, and that a differentiable function with a bounded
/// derivative is Lipschitz.

from order import lt_imp_lte, lt_trans, lte_antisymm, not_lt_imp_gte, lt_imp_ne_symm
from real.continuity_base import Real, continuous, uniform, uniform_condition, uniform_imp_continuous
from real.derivative_basic import has_derivative_at, has_derivative_at_unique, sub_ne_zero_of_ne
from real.calculus_api import is_derivative_fn, is_derivative_fn_at
from real.mean_value import mean_value_theorem, secant_slope
from real.real_base import abs_gte_zero, pos_gt_zero, gt_zero_imp_pos, lte_lt_trans, pos_imp_eq_abs
from real.real_seq import lt_imp_minus_pos
from real.real_ring import mul_zero_left, real_mul_comm
from real.exp import abs_div
from real.real_ring import exists_small_mul_variant_2
from ordered_field import mul_lt_mul_of_pos_right, multiply_inequality_with_nonnegative_element
from real.derivative_continuity import div_mul_cancel_denominator

numerals Real

/// True if f is Lipschitz continuous with constant k: k is positive and the
/// value difference is bounded by k times the argument difference.
define lipschitz_with(f: Real -> Real, k: Real) -> Bool {
    Real.0 < k and forall(x: Real, y: Real) {
        (f(x) - f(y)).abs <= k * (x - y).abs
    }
}

/// True if f is a Lipschitz continuous function.
define is_lipschitz(f: Real -> Real) -> Bool {
    exists(k: Real) {
        lipschitz_with(f, k)
    }
}

/// A Lipschitz function has a positive constant.
theorem lipschitz_with_pos_const(f: Real -> Real, k: Real) {
    lipschitz_with(f, k) implies Real.0 < k
} by {
    if lipschitz_with(f, k) {
        lipschitz_with(f, k) = (Real.0 < k and forall(x: Real, y: Real) {
            (f(x) - f(y)).abs <= k * (x - y).abs
        })
        Real.0 < k and forall(x: Real, y: Real) {
            (f(x) - f(y)).abs <= k * (x - y).abs
        }
        Real.0 < k
    }
}

/// Applying the Lipschitz bound to a pair of arguments.
theorem lipschitz_with_apply(f: Real -> Real, k: Real, x0: Real, y0: Real) {
    lipschitz_with(f, k) implies (f(x0) - f(y0)).abs <= k * (x0 - y0).abs
} by {
    if lipschitz_with(f, k) {
        lipschitz_with(f, k) = (Real.0 < k and forall(x: Real, y: Real) {
            (f(x) - f(y)).abs <= k * (x - y).abs
        })
        forall(x1: Real, y1: Real) {
            (f(x1) - f(y1)).abs <= k * (x1 - y1).abs
        }
        (f(x0) - f(y0)).abs <= k * (x0 - y0).abs
    }
}

/// A Lipschitz function is uniformly continuous.
theorem lipschitz_imp_uniform(f: Real -> Real, k: Real) {
    lipschitz_with(f, k) implies uniform(f)
} by {
    if lipschitz_with(f, k) {
        forall(eps: Real) {
            if eps.is_positive {
                lipschitz_with_pos_const(f, k)
                Real.0 < k
                gt_zero_imp_pos(k)
                k.is_positive
                exists_small_mul_variant_2(k, eps)
                let delta: Real satisfy {
                    delta.is_positive and delta * k < eps
                }
                forall(x: Real, y: Real) {
                    if x.is_close(y, delta) {
                        x.is_close(y, delta) = (x - y).abs < delta
                        (x - y).abs < delta
                        lipschitz_with_apply(f, k, x, y)
                        (f(x) - f(y)).abs <= k * (x - y).abs
                        gt_zero_imp_pos(k)
                        k.is_positive
                        mul_lt_mul_of_pos_right[Real]((x - y).abs, delta, k)
                        (x - y).abs * k < delta * k
                        real_mul_comm((x - y).abs, k)
                        (x - y).abs * k = k * (x - y).abs
                        k * (x - y).abs < delta * k
                        lt_trans[Real](k * (x - y).abs, delta * k, eps)
                        k * (x - y).abs < eps
                        lte_lt_trans((f(x) - f(y)).abs, k * (x - y).abs, eps)
                        (f(x) - f(y)).abs < eps
                        f(x).is_close(f(y), eps)
                    }
                }
                uniform_condition(f, delta, eps)
                delta.is_positive and uniform_condition(f, delta, eps)
                exists(delta2: Real) {
                    delta2.is_positive and uniform_condition(f, delta2, eps)
                }
            }
        }
    }
}

/// A Lipschitz function is continuous.
theorem lipschitz_imp_continuous(f: Real -> Real, k: Real) {
    lipschitz_with(f, k) implies continuous(f)
} by {
    if lipschitz_with(f, k) {
        lipschitz_imp_uniform(f, k)
        uniform(f)
        uniform_imp_continuous(f)
        continuous(f)
    }
}

/// A differentiable function whose derivative is bounded by k on an interval
/// is k-Lipschitz there.
theorem derivative_bound_imp_lipschitz(
    f: Real -> Real, df: Real -> Real, k: Real, a: Real, b: Real
) {
    is_derivative_fn(f, df) and (forall(z: Real) { df(z).abs <= k }) and
    continuous(f) and Real.0 <= k and a <= b
    implies (f(b) - f(a)).abs <= k * (b - a).abs
} by {
    if is_derivative_fn(f, df) and (forall(z: Real) { df(z).abs <= k }) and
       continuous(f) and Real.0 <= k and a <= b {
        if a < b {
            mean_value_theorem(f, df, a, b)
            let c: Real satisfy {
                a < c and c < b and has_derivative_at(f, c, secant_slope(f, a, b))
            }
            has_derivative_at(f, c, secant_slope(f, a, b))
            is_derivative_fn_at(f, df, c)
            has_derivative_at(f, c, df(c))
            has_derivative_at_unique(f, c, secant_slope(f, a, b), df(c))
            secant_slope(f, a, b) = df(c)
            secant_slope(f, a, b) = (f(b) - f(a)) / (b - a)
            df(c) = (f(b) - f(a)) / (b - a)
            forall(z: Real) { df(z).abs <= k }
            df(c).abs <= k
            abs_div(f(b) - f(a), b - a)
            ((f(b) - f(a)) / (b - a)).abs = (f(b) - f(a)).abs / (b - a).abs
            df(c).abs = (f(b) - f(a)).abs / (b - a).abs
            (f(b) - f(a)).abs / (b - a).abs <= k
            lt_imp_minus_pos(a, b)
            (b - a).is_positive
            pos_imp_eq_abs(b - a)
            b - a = (b - a).abs
            (b - a).abs = b - a
            (f(b) - f(a)).abs / (b - a) <= k
            pos_gt_zero(b - a)
            b - a > Real.0
            multiply_inequality_with_nonnegative_element[Real](
                (f(b) - f(a)).abs / (b - a), k, b - a)
            ((f(b) - f(a)).abs / (b - a)) * (b - a) <= k * (b - a)
            lt_imp_ne_symm(a, b)
            b != a
            sub_ne_zero_of_ne(b, a)
            b - a != Real.0
            div_mul_cancel_denominator((f(b) - f(a)).abs, b - a)
            ((f(b) - f(a)).abs / (b - a)) * (b - a) = (f(b) - f(a)).abs
            (f(b) - f(a)).abs <= k * (b - a)
            (b - a).abs = b - a
            k * (b - a) = k * (b - a).abs
            (f(b) - f(a)).abs <= k * (b - a).abs
        } else {
            not_lt_imp_gte[Real](a, b)
            a >= b
            b <= a
            lte_antisymm[Real](a, b)
            a = b
            b = a
            f(b) = f(a)
            f(b) - f(a) = Real.0
            (f(b) - f(a)).abs = Real.0.abs
            Real.0.abs = Real.0
            (f(b) - f(a)).abs = Real.0
            abs_gte_zero(b - a)
            Real.0 <= (b - a).abs
            multiply_inequality_with_nonnegative_element[Real](Real.0, k, (b - a).abs)
            Real.0 * (b - a).abs <= k * (b - a).abs
            mul_zero_left((b - a).abs)
            Real.0 * (b - a).abs = Real.0
            Real.0 <= k * (b - a).abs
            (f(b) - f(a)).abs <= k * (b - a).abs
        }
    }
}
