from data.basic.set import Set, empty_set_contains_eq, universal_set_contains_eq,
    subset_contains, sets_subset_union, sets_subset_contain_union, sets_subset_intersection,
    intersection_contains_intro,
    union_contains_eq, union_contains_left, union_contains_right, double_inclusion,
    subset_contains_eq
from real.real_field import Real
from real.real_base import self_close

numerals Real

/// True if `x` is `eps`-adherent to a set of real numbers.
define is_eps_adherent_to_set(s: Set[Real], x: Real, eps: Real) -> Bool {
    exists(y: Real) {
        s.contains(y) and y.is_close(x, eps)
    }
}

/// True if `x` is an adherent point of a set of real numbers.
define is_adherent_point_of_set(s: Set[Real], x: Real) -> Bool {
    forall(eps: Real) {
        eps.is_positive implies is_eps_adherent_to_set(s, x, eps)
    }
}

/// True if `x` lies in the closure of `s`.
define in_closure(s: Set[Real], x: Real) -> Bool {
    is_adherent_point_of_set(s, x)
}

/// The closure of a set of real numbers.
define closure(s: Set[Real]) -> Set[Real] {
    Set[Real].new(in_closure(s))
}

/// True if a set of real numbers contains all of its adherent points.
define is_closed_set(s: Set[Real]) -> Bool {
    forall(x: Real) {
        is_adherent_point_of_set(s, x) implies s.contains(x)
    }
}

/// True if `x` is a limit point of a set of real numbers.
define is_limit_point_of_set(s: Set[Real], x: Real) -> Bool {
    is_adherent_point_of_set(s.difference(Set[Real].singleton(x)), x)
}

/// True if `x` is an isolated point of a set of real numbers.
define is_isolated_point_of_set(s: Set[Real], x: Real) -> Bool {
    s.contains(x) and exists(eps: Real) {
        eps.is_positive and forall(y: Real) {
            s.contains(y) and y != x implies not y.is_close(x, eps)
        }
    }
}

/// True if a set of real numbers is contained in a symmetric bounded interval.
define is_bounded_real_set(s: Set[Real]) -> Bool {
    exists(bound: Real) {
        bound > Real.0 and forall(x: Real) {
            s.contains(x) implies -bound <= x and x <= bound
        }
    }
}

/// True if `x` is an interior point of a set of real numbers.
define is_interior_point(s: Set[Real], x: Real) -> Bool {
    s.contains(x) and exists(eps: Real) {
        eps.is_positive and forall(y: Real) {
            y.is_close(x, eps) implies s.contains(y)
        }
    }
}

/// True if `x` lies in the interior of `s`.
define in_interior(s: Set[Real], x: Real) -> Bool {
    is_interior_point(s, x)
}

/// The interior of a set of real numbers.
define interior(s: Set[Real]) -> Set[Real] {
    Set[Real].new(in_interior(s))
}

/// True if every point of a set of real numbers is an interior point.
define is_open_set(s: Set[Real]) -> Bool {
    forall(x: Real) {
        s.contains(x) implies is_interior_point(s, x)
    }
}

/// True if a triple witnesses a failure of interval convexity in `s`.
define disconnecting_triple(s: Set[Real], x: Real, y: Real, z: Real) -> Bool {
    s.contains(x) and s.contains(y) and x < y and x <= z and z <= y and not s.contains(z)
}

/// True if a set of real numbers contains every point between any two of its points.
define is_connected_real_set(s: Set[Real]) -> Bool {
    forall(x: Real, y: Real, z: Real) {
        not disconnecting_triple(s, x, y, z)
    }
}

/// Membership in a closure is adherence.
theorem closure_contains_eq(s: Set[Real], x: Real) {
    closure(s).contains(x) = is_adherent_point_of_set(s, x)
}

/// Membership in an interior is interior-point membership.
theorem interior_contains_eq(s: Set[Real], x: Real) {
    interior(s).contains(x) = is_interior_point(s, x)
}

/// A point is adherent if it is epsilon-adherent for every positive epsilon.
theorem adherent_point_intro(s: Set[Real], x: Real) {
    forall(eps: Real) {
        eps.is_positive implies is_eps_adherent_to_set(s, x, eps)
    } implies is_adherent_point_of_set(s, x)
}

/// An adherent point is epsilon-adherent for every positive epsilon.
theorem adherent_point_eps(s: Set[Real], x: Real, eps: Real) {
    is_adherent_point_of_set(s, x) and eps.is_positive
    implies is_eps_adherent_to_set(s, x, eps)
}

/// An adherent point lies in the closure.
theorem closure_contains_of_adherent(s: Set[Real], x: Real) {
    is_adherent_point_of_set(s, x) implies closure(s).contains(x)
} by {
    if is_adherent_point_of_set(s, x) {
        closure_contains_eq(s, x)
    }
}

/// A point in the closure is adherent.
theorem closure_contains_imp_adherent(s: Set[Real], x: Real) {
    closure(s).contains(x) implies is_adherent_point_of_set(s, x)
} by {
    if closure(s).contains(x) {
        closure_contains_eq(s, x)
    }
}

/// A closed set contains each of its adherent points.
theorem closed_set_contains_adherent(s: Set[Real], x: Real) {
    is_closed_set(s) and is_adherent_point_of_set(s, x) implies s.contains(x)
} by {
    if is_closed_set(s) and is_adherent_point_of_set(s, x) {
        is_closed_set(s) = forall(y: Real) {
            is_adherent_point_of_set(s, y) implies s.contains(y)
        }
        forall(y: Real) {
            is_adherent_point_of_set(s, y) implies s.contains(y)
        }
        is_adherent_point_of_set(s, x) implies s.contains(x)
        s.contains(x)
    }
}

/// A positive ball contained in a set makes its center an interior point.
theorem interior_point_intro(s: Set[Real], x: Real, eps: Real) {
    s.contains(x) and eps.is_positive and forall(y: Real) {
        y.is_close(x, eps) implies s.contains(y)
    } implies is_interior_point(s, x)
} by {
    if s.contains(x) and eps.is_positive and forall(y: Real) {
        y.is_close(x, eps) implies s.contains(y)
    } {
        exists(e: Real) {
            e.is_positive and forall(y: Real) {
                y.is_close(x, e) implies s.contains(y)
            }
        }
    }
}

/// An open set makes each of its points an interior point.
theorem open_set_interior_point(s: Set[Real], x: Real) {
    is_open_set(s) and s.contains(x) implies is_interior_point(s, x)
} by {
    if is_open_set(s) and s.contains(x) {
        is_open_set(s) = forall(y: Real) {
            s.contains(y) implies is_interior_point(s, y)
        }
        forall(y: Real) {
            s.contains(y) implies is_interior_point(s, y)
        }
        s.contains(x) implies is_interior_point(s, x)
        is_interior_point(s, x)
    }
}

/// A disconnecting triple starts with a point of the set.
theorem disconnecting_triple_contains_left(s: Set[Real], x: Real, y: Real, z: Real) {
    disconnecting_triple(s, x, y, z) implies s.contains(x)
} by {
    if disconnecting_triple(s, x, y, z) {
        disconnecting_triple(s, x, y, z) =
            (s.contains(x) and s.contains(y) and x < y and x <= z and z <= y and not s.contains(z))
        s.contains(x)
    }
}

/// A disconnecting triple omits its middle witness from the set.
theorem disconnecting_triple_not_contains_middle(s: Set[Real], x: Real, y: Real, z: Real) {
    disconnecting_triple(s, x, y, z) implies not s.contains(z)
} by {
    if disconnecting_triple(s, x, y, z) {
        disconnecting_triple(s, x, y, z) =
            (s.contains(x) and s.contains(y) and x < y and x <= z and z <= y and not s.contains(z))
        not s.contains(z)
    }
}

/// A close member of a set witnesses epsilon-adherence to that set.
theorem eps_adherent_of_contains_close(s: Set[Real], x: Real, y: Real, eps: Real) {
    s.contains(y) and y.is_close(x, eps) implies is_eps_adherent_to_set(s, x, eps)
} by {
    if s.contains(y) and y.is_close(x, eps) {
        exists(z: Real) {
            s.contains(z) and z.is_close(x, eps)
        }
    }
}

/// An epsilon-adherent point has a close member witness.
theorem eps_adherent_witness(s: Set[Real], x: Real, eps: Real) {
    is_eps_adherent_to_set(s, x, eps) implies exists(y: Real) {
        s.contains(y) and y.is_close(x, eps)
    }
}

/// Epsilon-adherence is monotone with respect to subset inclusion.
theorem eps_adherent_of_subset(s: Set[Real], t: Set[Real], x: Real, eps: Real) {
    s.subset(t) and is_eps_adherent_to_set(s, x, eps)
    implies is_eps_adherent_to_set(t, x, eps)
} by {
    if s.subset(t) and is_eps_adherent_to_set(s, x, eps) {
        let y: Real satisfy {
            s.contains(y) and y.is_close(x, eps)
        }
        subset_contains(s, t, y)
        t.contains(y)
        eps_adherent_of_contains_close(t, x, y, eps)
        is_eps_adherent_to_set(t, x, eps)
    }
}

/// Every member of a set is an adherent point of that set.
theorem point_in_set_is_adherent(s: Set[Real], x: Real) {
    s.contains(x) implies is_adherent_point_of_set(s, x)
} by {
    if s.contains(x) {
        forall(eps: Real) {
            if eps.is_positive {
                self_close(x, eps)
                x.is_close(x, eps)
                eps_adherent_of_contains_close(s, x, x, eps)
                is_eps_adherent_to_set(s, x, eps)
            }
        }
    }
}

/// Every member of a set lies in its closure.
theorem point_in_closure_if_in_set(s: Set[Real], x: Real) {
    s.contains(x) implies closure(s).contains(x)
} by {
    if s.contains(x) {
        point_in_set_is_adherent(s, x)
        closure_contains_eq(s, x)
    }
}

/// A set is contained in its closure.
theorem set_subset_closure(s: Set[Real]) {
    s.subset(closure(s))
} by {
    forall(x: Real) {
        if s.contains(x) {
            point_in_closure_if_in_set(s, x)
        }
    }
}

/// The closure of a closed set is contained in the set.
theorem closure_subset_of_closed_set(s: Set[Real]) {
    is_closed_set(s) implies closure(s).subset(s)
} by {
    if is_closed_set(s) {
        forall(x: Real) {
            if closure(s).contains(x) {
                closure_contains_eq(s, x)
                is_adherent_point_of_set(s, x)
                closed_set_contains_adherent(s, x)
            }
        }
        closure(s).subset(s)
    }
}

/// A closed set is equal to its closure.
theorem closed_set_eq_closure(s: Set[Real]) {
    is_closed_set(s) implies s = closure(s)
} by {
    if is_closed_set(s) {
        set_subset_closure(s)
        closure_subset_of_closed_set(s)
        s.subset(closure(s)) and closure(s).subset(s)
        double_inclusion(s, closure(s))
    }
}

/// Subset inclusion transports closure membership.
theorem closure_contains_of_subset(s: Set[Real], t: Set[Real], x: Real) {
    s.subset(t) and closure(s).contains(x) implies closure(t).contains(x)
} by {
    if s.subset(t) and closure(s).contains(x) {
        closure_contains_eq(s, x)
        is_adherent_point_of_set(s, x)
        forall(eps: Real) {
            if eps.is_positive {
                adherent_point_eps(s, x, eps)
                is_eps_adherent_to_set(s, x, eps)
                eps_adherent_of_subset(s, t, x, eps)
                is_eps_adherent_to_set(t, x, eps)
            }
        }
        adherent_point_intro(t, x)
        is_adherent_point_of_set(t, x)
        closure_contains_eq(t, x)
        closure(t).contains(x)
    }
}

/// Closure is monotone with respect to subset inclusion.
theorem closure_mono(s: Set[Real], t: Set[Real]) {
    s.subset(t) implies closure(s).subset(closure(t))
} by {
    if s.subset(t) {
        forall(x: Real) {
            if closure(s).contains(x) {
                closure_contains_of_subset(s, t, x)
            }
        }
        subset_contains_eq[Real](closure(s), closure(t))
        closure(s).subset(closure(t)) = forall(x: Real) {
            closure(s).contains(x) implies closure(t).contains(x)
        }
        closure(s).subset(closure(t))
    }
}

/// The closure of an intersection is contained in the intersection of closures.
theorem closure_intersection_subset(s: Set[Real], t: Set[Real]) {
    closure(s.intersection(t)).subset(closure(s).intersection(closure(t)))
} by {
    let cst = closure(s.intersection(t))
    let cs = closure(s)
    let ct = closure(t)
    let csi = cs.intersection(ct)
    sets_subset_intersection[Real](s, t)
    s.intersection(t).subset(s)
    s.intersection(t).subset(t)
    closure_mono(s.intersection(t), s)
    closure(s.intersection(t)).subset(closure(s))
    closure_mono(s.intersection(t), t)
    closure(s.intersection(t)).subset(closure(t))
    forall(x: Real) {
        if cst.contains(x) {
            cs.contains(x)
            ct.contains(x)
            intersection_contains_intro(cs, ct, x)
            csi.contains(x)
        }
    }
    subset_contains_eq[Real](cst, csi)
    cst.subset(csi) = forall(x: Real) {
        cst.contains(x) implies csi.contains(x)
    }
    cst.subset(csi)
}

/// The union of closures is contained in the closure of the union.
theorem union_closure_subset_closure_union(s: Set[Real], t: Set[Real]) {
    closure(s).union(closure(t)).subset(closure(s.union(t)))
} by {
    sets_subset_union[Real](s, t)
    s.subset(s.union(t))
    t.subset(s.union(t))
    closure_mono(s, s.union(t))
    closure(s).subset(closure(s.union(t)))
    closure_mono(t, s.union(t))
    closure(t).subset(closure(s.union(t)))
    sets_subset_contain_union[Real](closure(s), closure(t), closure(s.union(t)))
}

/// A union is contained in the union of the closures.
theorem union_subset_union_closure(s: Set[Real], t: Set[Real]) {
    s.union(t).subset(closure(s).union(closure(t)))
} by {
    set_subset_closure(s)
    set_subset_closure(t)
    forall(x: Real) {
        if s.union(t).contains(x) {
            union_contains_eq(s, t, x)
            if s.contains(x) {
                closure(s).contains(x)
                union_contains_left(closure(s), closure(t), x)
            } else {
                t.contains(x)
                closure(t).contains(x)
                union_contains_right(closure(s), closure(t), x)
            }
        }
    }
    s.union(t).subset(closure(s).union(closure(t)))
}

/// The interior of a set is contained in the set.
theorem interior_subset(s: Set[Real]) {
    interior(s).subset(s)
} by {
    forall(x: Real) {
        if interior(s).contains(x) {
            interior_contains_eq(s, x)
            is_interior_point(s, x)
            s.contains(x)
        }
    }
}

/// Every interior point of a set is an adherent point of that set.
theorem interior_point_is_adherent(s: Set[Real], x: Real) {
    is_interior_point(s, x) implies is_adherent_point_of_set(s, x)
} by {
    if is_interior_point(s, x) {
        s.contains(x)
        point_in_set_is_adherent(s, x)
    }
}

/// The empty set is open.
theorem empty_set_is_open {
    is_open_set(Set[Real].empty_set)
} by {
    forall(x: Real) {
        if Set[Real].empty_set.contains(x) {
            empty_set_contains_eq[Real](x)
            false
        }
    }
}

/// The empty set is closed.
theorem empty_set_is_closed {
    is_closed_set(Set[Real].empty_set)
} by {
    forall(x: Real) {
        if is_adherent_point_of_set(Set[Real].empty_set, x) {
            Real.1.is_positive
            is_eps_adherent_to_set(Set[Real].empty_set, x, Real.1)
            let y: Real satisfy {
                Set[Real].empty_set.contains(y) and y.is_close(x, Real.1)
            }
            empty_set_contains_eq[Real](y)
            false
        }
    }
}

/// The real line is open.
theorem universal_set_is_open {
    is_open_set(Set[Real].universal_set)
} by {
    forall(x: Real) {
        if Set[Real].universal_set.contains(x) {
            Real.1.is_positive
            forall(y: Real) {
                if y.is_close(x, Real.1) {
                    universal_set_contains_eq[Real](y)
                    Set[Real].universal_set.contains(y)
                }
            }
            exists(eps: Real) {
                eps.is_positive and forall(y: Real) {
                    y.is_close(x, eps) implies Set[Real].universal_set.contains(y)
                }
            }
            is_interior_point(Set[Real].universal_set, x)
        }
    }
}

/// The real line is closed.
theorem universal_set_is_closed {
    is_closed_set(Set[Real].universal_set)
} by {
    forall(x: Real) {
        if is_adherent_point_of_set(Set[Real].universal_set, x) {
            universal_set_contains_eq[Real](x)
            Set[Real].universal_set.contains(x)
        }
    }
}

/// The empty set is connected.
theorem empty_set_is_connected {
    is_connected_real_set(Set[Real].empty_set)
} by {
    let e = Set[Real].empty_set
    forall(x: Real, y: Real, z: Real) {
        if disconnecting_triple(e, x, y, z) {
            disconnecting_triple_contains_left(e, x, y, z)
            empty_set_contains_eq[Real](x)
            false
        }
    }
}

/// The real line is connected.
theorem universal_set_is_connected {
    is_connected_real_set(Set[Real].universal_set)
} by {
    let u = Set[Real].universal_set
    forall(x: Real, y: Real, z: Real) {
        if disconnecting_triple(u, x, y, z) {
            disconnecting_triple_not_contains_middle(u, x, y, z)
            universal_set_contains_eq[Real](z)
            u.contains(z)
            false
        }
    }
}

/// A symmetric bound gives a lower bound for every member of the set.
theorem bounded_real_set_lower_bound(s: Set[Real], bound: Real, x: Real) {
    forall(y: Real) {
        s.contains(y) implies -bound <= y and y <= bound
    } and s.contains(x) implies -bound <= x
}

/// A symmetric bound gives an upper bound for every member of the set.
theorem bounded_real_set_upper_bound(s: Set[Real], bound: Real, x: Real) {
    forall(y: Real) {
        s.contains(y) implies -bound <= y and y <= bound
    } and s.contains(x) implies x <= bound
}

/// Every subset of a bounded real set is bounded.
theorem subset_of_bounded_real_set_is_bounded(s: Set[Real], t: Set[Real]) {
    s.subset(t) and is_bounded_real_set(t) implies is_bounded_real_set(s)
} by {
    if s.subset(t) and is_bounded_real_set(t) {
        let bound: Real satisfy {
            bound > Real.0 and forall(x: Real) {
                t.contains(x) implies -bound <= x and x <= bound
            }
        }
        forall(x: Real) {
            if s.contains(x) {
                t.contains(x)
                bounded_real_set_lower_bound(t, bound, x)
                bounded_real_set_upper_bound(t, bound, x)
                -bound <= x and x <= bound
            }
        }
        bound > Real.0 and forall(x: Real) {
            s.contains(x) implies -bound <= x and x <= bound
        }
    }
}

/// The empty set of real numbers is bounded.
theorem empty_real_set_is_bounded {
    is_bounded_real_set(Set[Real].empty_set)
} by {
    Real.1 > Real.0
    forall(x: Real) {
        if Set[Real].empty_set.contains(x) {
            empty_set_contains_eq[Real](x)
            false
        }
    }
    exists(bound: Real) {
        bound > Real.0 and forall(x: Real) {
            Set[Real].empty_set.contains(x) implies -bound <= x and x <= bound
        }
    }
}

/// A singleton set of real numbers is bounded.
theorem singleton_real_set_is_bounded(a: Real) {
    is_bounded_real_set(Set[Real].singleton(a))
} by {
    let bound = a.abs + Real.1
    not a.abs.is_negative
    a.abs >= Real.0
    a.abs + Real.1 >= Real.0 + Real.1
    Real.0 + Real.1 = Real.1
    a.abs + Real.1 >= Real.1
    Real.1 > Real.0
    bound > Real.0
    forall(x: Real) {
        if Set[Real].singleton(a).contains(x) {
            x = a
            -a.abs <= a
            a <= a.abs
            a.abs < bound
            a.abs <= bound
            -bound <= -a.abs
            -bound <= a
            a <= bound
            -bound <= x and x <= bound
        }
    }
    exists(b: Real) {
        b > Real.0 and forall(x: Real) {
            Set[Real].singleton(a).contains(x) implies -b <= x and x <= b
        }
    }
}

/// The union of two bounded real sets is bounded.
theorem union_of_bounded_real_sets_is_bounded(s: Set[Real], t: Set[Real]) {
    is_bounded_real_set(s) and is_bounded_real_set(t) implies is_bounded_real_set(s.union(t))
} by {
    if is_bounded_real_set(s) and is_bounded_real_set(t) {
        let bs: Real satisfy {
            bs > Real.0 and forall(y: Real) {
                s.contains(y) implies -bs <= y and y <= bs
            }
        }
        let bt: Real satisfy {
            bt > Real.0 and forall(y: Real) {
                t.contains(y) implies -bt <= y and y <= bt
            }
        }
        let bound = bs.max(bt)
        bs <= bound
        bt <= bound
        bound > Real.0
        -bound <= -bs
        -bound <= -bt
        forall(x: Real) {
            if s.union(t).contains(x) {
                union_contains_eq(s, t, x)
                if s.contains(x) {
                    bounded_real_set_lower_bound(s, bs, x)
                    bounded_real_set_upper_bound(s, bs, x)
                    -bs <= x
                    x <= bs
                    -bound <= x
                    x <= bound
                    -bound <= x and x <= bound
                } else {
                    t.contains(x)
                    bounded_real_set_lower_bound(t, bt, x)
                    bounded_real_set_upper_bound(t, bt, x)
                    -bt <= x
                    x <= bt
                    -bound <= x
                    x <= bound
                    -bound <= x and x <= bound
                }
            }
        }
        exists(bound2: Real) {
            bound2 > Real.0 and forall(x: Real) {
                s.union(t).contains(x) implies -bound2 <= x and x <= bound2
            }
        }
    }
}
