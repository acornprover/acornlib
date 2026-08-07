from data.basic.function_algebra import pointwise_mul
from data.basic.functions import compose, function_eq_transport_predicate_rev
from real.continuity_base import Real, continuous_at
from real.continuity_algebra import compose_continuous_at
from real.continuity_pointwise_mul import continuous_at_pointwise_mul
from real.derivative_basic import has_derivative_at
from real.derivative_continuity import derivative_continuous_at
from real.derivative_quotient import reciprocal_real, pointwise_reciprocal_real,
    pointwise_div_real, pointwise_reciprocal_real_eq_compose,
    pointwise_div_real_eq_mul_reciprocal, derivative_reciprocal_real

/// The reciprocal real function is continuous at every nonzero point.
theorem continuous_at_reciprocal_real(x0: Real) {
    x0 != Real.0 implies continuous_at(reciprocal_real, x0)
} by {
    if x0 != Real.0 {
        derivative_reciprocal_real(x0)
        has_derivative_at(reciprocal_real, x0, -Real.1 / (x0 * x0))
        derivative_continuous_at(reciprocal_real, x0, -Real.1 / (x0 * x0))
        continuous_at(reciprocal_real, x0)
    }
}

/// Pointwise reciprocal preserves continuity at a point where the denominator is nonzero.
theorem continuous_at_pointwise_reciprocal_real(g: Real -> Real, x0: Real) {
    continuous_at(g, x0) and g(x0) != Real.0 implies
        continuous_at(pointwise_reciprocal_real(g), x0)
} by {
    define continuous_at_x(h: Real -> Real) -> Bool {
        continuous_at(h, x0)
    }
    if continuous_at(g, x0) and g(x0) != Real.0 {
        continuous_at_reciprocal_real(g(x0))
        continuous_at(reciprocal_real, g(x0))
        compose_continuous_at(reciprocal_real, g, x0)
        continuous_at(compose(reciprocal_real, g), x0)
        pointwise_reciprocal_real_eq_compose(g)
        continuous_at_x(compose(reciprocal_real, g))
        function_eq_transport_predicate_rev(
            continuous_at_x,
            pointwise_reciprocal_real(g),
            compose(reciprocal_real, g)
        )
        continuous_at(pointwise_reciprocal_real(g), x0)
    }
}

/// Pointwise quotient preserves continuity at a point where the denominator is nonzero.
theorem continuous_at_pointwise_div_real(f: Real -> Real, g: Real -> Real, x0: Real) {
    continuous_at(f, x0) and continuous_at(g, x0) and g(x0) != Real.0 implies
        continuous_at(pointwise_div_real(f, g), x0)
} by {
    define continuous_at_x(h: Real -> Real) -> Bool {
        continuous_at(h, x0)
    }
    if continuous_at(f, x0) and continuous_at(g, x0) and g(x0) != Real.0 {
        continuous_at_pointwise_reciprocal_real(g, x0)
        continuous_at(pointwise_reciprocal_real(g), x0)
        continuous_at_pointwise_mul(f, pointwise_reciprocal_real(g), x0)
        continuous_at(pointwise_mul(f, pointwise_reciprocal_real(g)), x0)
        pointwise_div_real_eq_mul_reciprocal(f, g)
        continuous_at_x(pointwise_mul(f, pointwise_reciprocal_real(g)))
        function_eq_transport_predicate_rev(
            continuous_at_x,
            pointwise_div_real(f, g),
            pointwise_mul(f, pointwise_reciprocal_real(g))
        )
        continuous_at(pointwise_div_real(f, g), x0)
    }
}
