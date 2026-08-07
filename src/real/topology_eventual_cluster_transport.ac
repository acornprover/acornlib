/// Cluster-set consumers of eventual sequence tail/subsequence transport.
///
/// This module intentionally adds no new eventual predicate API and no
/// asymptotic notation.  It packages shallow consequences for cluster sets by
/// combining accepted eventual transport with existing cluster, closure, and
/// compactness support.

from data.basic.functions import compose
from nat import Nat
from data.basic.set import Set
from real.real_field import Real
from real.topology import is_bounded_real_set, is_closed_set
from real.topology_compact import is_compact_real_set
from real.limits import tends_to_infinity, is_subsequence_index, subsequence
from real.sequence_set_membership import seq_eventually_in_real_set
from real.sequence_eventual_bridge import seq_eventually_equal_real,
    seq_eventually_equal_real_preserves_eventual_membership
from real.sequence_eventual_tail_transport import seq_eventually_in_real_set_compose_tends_to_infinity,
    seq_eventually_in_real_set_shift_add,
    seq_eventually_in_real_set_subsequence
from real.sequence_cluster_sets import cluster_set_subset_closed_set_of_eventually_in,
    sequence_cluster_set
from real.sequence_cluster_compact import sequence_cluster_set_subset_compact_of_eventually_in,
    sequence_cluster_set_is_bounded_of_eventually_in_compact

/// Reindexing along a map tending to infinity preserves eventual closed-set
/// containment of cluster sets.
theorem compose_tends_to_infinity_cluster_subset_closed_of_eventually_in(
    s: Set[Real],
    a: Nat -> Real,
    f: Nat -> Nat
) {
    seq_eventually_in_real_set(s, a) and tends_to_infinity(f) and is_closed_set(s)
    implies sequence_cluster_set(compose(a, f)).subset(s)
} by {
    if seq_eventually_in_real_set(s, a) and tends_to_infinity(f) and is_closed_set(s) {
        seq_eventually_in_real_set_compose_tends_to_infinity(s, a, f)
        seq_eventually_in_real_set(s, compose(a, f))
        cluster_set_subset_closed_set_of_eventually_in(compose(a, f), s)
        sequence_cluster_set(compose(a, f)).subset(s)
    }
}

/// Reindexing along a map tending to infinity preserves eventual compact-set
/// containment of cluster sets.
theorem compose_tends_to_infinity_cluster_subset_compact_of_eventually_in(
    s: Set[Real],
    a: Nat -> Real,
    f: Nat -> Nat
) {
    seq_eventually_in_real_set(s, a) and tends_to_infinity(f) and is_compact_real_set(s)
    implies sequence_cluster_set(compose(a, f)).subset(s)
} by {
    if seq_eventually_in_real_set(s, a) and tends_to_infinity(f) and is_compact_real_set(s) {
        seq_eventually_in_real_set_compose_tends_to_infinity(s, a, f)
        seq_eventually_in_real_set(s, compose(a, f))
        sequence_cluster_set_subset_compact_of_eventually_in(compose(a, f), s)
        sequence_cluster_set(compose(a, f)).subset(s)
    }
}

/// A cluster set of a reindexing along a map tending to infinity is bounded
/// when the original sequence is eventually in a compact set.
theorem compose_tends_to_infinity_cluster_bounded_of_eventually_in_compact(
    s: Set[Real],
    a: Nat -> Real,
    f: Nat -> Nat
) {
    seq_eventually_in_real_set(s, a) and tends_to_infinity(f) and is_compact_real_set(s)
    implies is_bounded_real_set(sequence_cluster_set(compose(a, f)))
} by {
    if seq_eventually_in_real_set(s, a) and tends_to_infinity(f) and is_compact_real_set(s) {
        seq_eventually_in_real_set_compose_tends_to_infinity(s, a, f)
        seq_eventually_in_real_set(s, compose(a, f))
        sequence_cluster_set_is_bounded_of_eventually_in_compact(compose(a, f), s)
        is_bounded_real_set(sequence_cluster_set(compose(a, f)))
    }
}

/// Discarding a finite prefix preserves eventual closed-set containment of
/// cluster sets.
theorem shift_add_cluster_subset_closed_of_eventually_in(
    s: Set[Real],
    a: Nat -> Real,
    k: Nat
) {
    seq_eventually_in_real_set(s, a) and is_closed_set(s)
    implies sequence_cluster_set(compose(a, k.add)).subset(s)
} by {
    if seq_eventually_in_real_set(s, a) and is_closed_set(s) {
        seq_eventually_in_real_set_shift_add(s, a, k)
        seq_eventually_in_real_set(s, compose(a, k.add))
        cluster_set_subset_closed_set_of_eventually_in(compose(a, k.add), s)
        sequence_cluster_set(compose(a, k.add)).subset(s)
    }
}

/// Discarding a finite prefix preserves eventual compact-set containment of
/// cluster sets.
theorem shift_add_cluster_subset_compact_of_eventually_in(
    s: Set[Real],
    a: Nat -> Real,
    k: Nat
) {
    seq_eventually_in_real_set(s, a) and is_compact_real_set(s)
    implies sequence_cluster_set(compose(a, k.add)).subset(s)
} by {
    if seq_eventually_in_real_set(s, a) and is_compact_real_set(s) {
        seq_eventually_in_real_set_shift_add(s, a, k)
        seq_eventually_in_real_set(s, compose(a, k.add))
        sequence_cluster_set_subset_compact_of_eventually_in(compose(a, k.add), s)
        sequence_cluster_set(compose(a, k.add)).subset(s)
    }
}

/// A finite-prefix shift has bounded cluster set when the original sequence is
/// eventually in a compact set.
theorem shift_add_cluster_bounded_of_eventually_in_compact(
    s: Set[Real],
    a: Nat -> Real,
    k: Nat
) {
    seq_eventually_in_real_set(s, a) and is_compact_real_set(s)
    implies is_bounded_real_set(sequence_cluster_set(compose(a, k.add)))
} by {
    if seq_eventually_in_real_set(s, a) and is_compact_real_set(s) {
        seq_eventually_in_real_set_shift_add(s, a, k)
        seq_eventually_in_real_set(s, compose(a, k.add))
        sequence_cluster_set_is_bounded_of_eventually_in_compact(compose(a, k.add), s)
        is_bounded_real_set(sequence_cluster_set(compose(a, k.add)))
    }
}

/// Selected subsequences preserve eventual closed-set containment of cluster
/// sets.
theorem subsequence_cluster_subset_closed_of_eventually_in(
    s: Set[Real],
    a: Nat -> Real,
    f: Nat -> Nat
) {
    seq_eventually_in_real_set(s, a) and is_subsequence_index(f) and is_closed_set(s)
    implies sequence_cluster_set(subsequence(a, f)).subset(s)
} by {
    if seq_eventually_in_real_set(s, a) and is_subsequence_index(f) and is_closed_set(s) {
        seq_eventually_in_real_set_subsequence(s, a, f)
        seq_eventually_in_real_set(s, subsequence(a, f))
        cluster_set_subset_closed_set_of_eventually_in(subsequence(a, f), s)
        sequence_cluster_set(subsequence(a, f)).subset(s)
    }
}

/// Selected subsequences preserve eventual compact-set containment of cluster
/// sets.
theorem subsequence_cluster_subset_compact_of_eventually_in(
    s: Set[Real],
    a: Nat -> Real,
    f: Nat -> Nat
) {
    seq_eventually_in_real_set(s, a) and is_subsequence_index(f) and is_compact_real_set(s)
    implies sequence_cluster_set(subsequence(a, f)).subset(s)
} by {
    if seq_eventually_in_real_set(s, a) and is_subsequence_index(f) and is_compact_real_set(s) {
        seq_eventually_in_real_set_subsequence(s, a, f)
        seq_eventually_in_real_set(s, subsequence(a, f))
        sequence_cluster_set_subset_compact_of_eventually_in(subsequence(a, f), s)
        sequence_cluster_set(subsequence(a, f)).subset(s)
    }
}

/// A selected subsequence has bounded cluster set when the original sequence is
/// eventually in a compact set.
theorem subsequence_cluster_bounded_of_eventually_in_compact(
    s: Set[Real],
    a: Nat -> Real,
    f: Nat -> Nat
) {
    seq_eventually_in_real_set(s, a) and is_subsequence_index(f) and is_compact_real_set(s)
    implies is_bounded_real_set(sequence_cluster_set(subsequence(a, f)))
} by {
    if seq_eventually_in_real_set(s, a) and is_subsequence_index(f) and is_compact_real_set(s) {
        seq_eventually_in_real_set_subsequence(s, a, f)
        seq_eventually_in_real_set(s, subsequence(a, f))
        sequence_cluster_set_is_bounded_of_eventually_in_compact(subsequence(a, f), s)
        is_bounded_real_set(sequence_cluster_set(subsequence(a, f)))
    }
}

/// Eventual equality transports eventual closed-set containment to the cluster
/// set of the eventually equal sequence.
theorem eventually_equal_cluster_subset_closed_of_eventually_in(
    s: Set[Real],
    a: Nat -> Real,
    b: Nat -> Real
) {
    seq_eventually_in_real_set(s, a) and seq_eventually_equal_real(a, b) and is_closed_set(s)
    implies sequence_cluster_set(b).subset(s)
} by {
    if seq_eventually_in_real_set(s, a) and seq_eventually_equal_real(a, b) and is_closed_set(s) {
        seq_eventually_equal_real_preserves_eventual_membership(s, a, b)
        seq_eventually_in_real_set(s, b)
        cluster_set_subset_closed_set_of_eventually_in(b, s)
        sequence_cluster_set(b).subset(s)
    }
}

/// Eventual equality transports eventual compact-set containment to the cluster
/// set of the eventually equal sequence.
theorem eventually_equal_cluster_subset_compact_of_eventually_in(
    s: Set[Real],
    a: Nat -> Real,
    b: Nat -> Real
) {
    seq_eventually_in_real_set(s, a) and seq_eventually_equal_real(a, b) and is_compact_real_set(s)
    implies sequence_cluster_set(b).subset(s)
} by {
    if seq_eventually_in_real_set(s, a) and seq_eventually_equal_real(a, b) and is_compact_real_set(s) {
        seq_eventually_equal_real_preserves_eventual_membership(s, a, b)
        seq_eventually_in_real_set(s, b)
        sequence_cluster_set_subset_compact_of_eventually_in(b, s)
        sequence_cluster_set(b).subset(s)
    }
}

/// Eventual equality transports compact eventual containment to boundedness of
/// the eventually equal sequence's cluster set.
theorem eventually_equal_cluster_bounded_of_eventually_in_compact(
    s: Set[Real],
    a: Nat -> Real,
    b: Nat -> Real
) {
    seq_eventually_in_real_set(s, a) and seq_eventually_equal_real(a, b) and is_compact_real_set(s)
    implies is_bounded_real_set(sequence_cluster_set(b))
} by {
    if seq_eventually_in_real_set(s, a) and seq_eventually_equal_real(a, b) and is_compact_real_set(s) {
        seq_eventually_equal_real_preserves_eventual_membership(s, a, b)
        seq_eventually_in_real_set(s, b)
        sequence_cluster_set_is_bounded_of_eventually_in_compact(b, s)
        is_bounded_real_set(sequence_cluster_set(b))
    }
}
