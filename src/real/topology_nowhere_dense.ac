from data.basic.set import Set, double_inclusion, empty_set_is_always_subset
from real.real_field import Real
from real.topology import closure, closure_mono, interior
from real.topology_closure import closure_idempotent
from real.topology_codense import is_codense_real_set
from real.topology_dense import is_dense_real_set
from real.topology_interior_algebra import interior_mono
from real.topology_interior_closure_idempotent import closure_empty_real_set,
    interior_empty_real_set

/// True if the closure of a real set has empty interior.
define is_nowhere_dense_real_set(s: Set[Real]) -> Bool {
    interior(closure(s)) = Set[Real].empty_set
}

/// Nowhere density is empty interior of the closure.
theorem nowhere_dense_real_set_eq_empty_interior_closure(s: Set[Real]) {
    is_nowhere_dense_real_set(s) = (interior(closure(s)) = Set[Real].empty_set)
}

/// The empty real set is nowhere dense.
theorem empty_real_set_is_nowhere_dense {
    is_nowhere_dense_real_set(Set[Real].empty_set)
} by {
    closure_empty_real_set
    closure(Set[Real].empty_set) = Set[Real].empty_set
    interior_empty_real_set
    interior(Set[Real].empty_set) = Set[Real].empty_set
    interior(closure(Set[Real].empty_set)) = Set[Real].empty_set
    is_nowhere_dense_real_set(Set[Real].empty_set)
}

/// A subset of a nowhere dense real set is nowhere dense.
theorem nowhere_dense_real_set_of_subset(s: Set[Real], t: Set[Real]) {
    s.subset(t) and is_nowhere_dense_real_set(t) implies is_nowhere_dense_real_set(s)
} by {
    if s.subset(t) and is_nowhere_dense_real_set(t) {
        closure_mono(s, t)
        closure(s).subset(closure(t))
        interior_mono(closure(s), closure(t))
        interior(closure(s)).subset(interior(closure(t)))
        interior(closure(t)) = Set[Real].empty_set
        interior(closure(s)).subset(Set[Real].empty_set)
        empty_set_is_always_subset[Real](interior(closure(s)))
        Set[Real].empty_set.subset(interior(closure(s)))
        double_inclusion(interior(closure(s)), Set[Real].empty_set)
        interior(closure(s)) = Set[Real].empty_set
        is_nowhere_dense_real_set(s)
    }
}

/// Nowhere density is unchanged by taking closure.
theorem nowhere_dense_closure_eq(s: Set[Real]) {
    is_nowhere_dense_real_set(closure(s)) = is_nowhere_dense_real_set(s)
} by {
    closure_idempotent(s)
    closure(closure(s)) = closure(s)
    if is_nowhere_dense_real_set(closure(s)) {
        interior(closure(closure(s))) = Set[Real].empty_set
        interior(closure(s)) = Set[Real].empty_set
        is_nowhere_dense_real_set(s)
    }
    if is_nowhere_dense_real_set(s) {
        interior(closure(s)) = Set[Real].empty_set
        interior(closure(closure(s))) = Set[Real].empty_set
        is_nowhere_dense_real_set(closure(s))
    }
    is_nowhere_dense_real_set(closure(s)) = is_nowhere_dense_real_set(s)
}

/// The closure of a nowhere dense real set is nowhere dense.
theorem closure_of_nowhere_dense_real_set_is_nowhere_dense(s: Set[Real]) {
    is_nowhere_dense_real_set(s) implies is_nowhere_dense_real_set(closure(s))
} by {
    if is_nowhere_dense_real_set(s) {
        nowhere_dense_closure_eq(s)
        is_nowhere_dense_real_set(closure(s)) = is_nowhere_dense_real_set(s)
        is_nowhere_dense_real_set(closure(s))
    }
}

/// A real set is nowhere dense when its closure is codense.
theorem nowhere_dense_of_codense_closure(s: Set[Real]) {
    is_codense_real_set(closure(s)) implies is_nowhere_dense_real_set(s)
} by {
    if is_codense_real_set(closure(s)) {
        from real.topology_codense import interior_of_codense_real_set_is_empty
        interior_of_codense_real_set_is_empty(closure(s))
        interior(closure(s)) = Set[Real].empty_set
        is_nowhere_dense_real_set(s)
    }
}

/// A nowhere dense real set has codense closure.
theorem codense_closure_of_nowhere_dense(s: Set[Real]) {
    is_nowhere_dense_real_set(s) implies is_codense_real_set(closure(s))
} by {
    if is_nowhere_dense_real_set(s) {
        interior(closure(s)) = Set[Real].empty_set
        from real.topology_closure_interior_duality import interior_eq_closure_complement_complement
        interior_eq_closure_complement_complement(closure(s))
        interior(closure(s)) = closure(closure(s).c).c
        closure(closure(s).c).c = Set[Real].empty_set
from data.basic.set import compl_of_compl_is_self
        closure(closure(s).c).c.c = Set[Real].empty_set.c
        compl_of_compl_is_self[Real](closure(closure(s).c))
        closure(closure(s).c).c.c = closure(closure(s).c)
from data.basic.set import empty_set_compl_is_universal
        empty_set_compl_is_universal[Real]
        Set[Real].empty_set.c = Set[Real].universal_set
        closure(closure(s).c) = Set[Real].universal_set
        is_dense_real_set(closure(s).c)
        is_codense_real_set(closure(s))
    }
}

/// Nowhere density is equivalent to codensity of the closure.
theorem nowhere_dense_eq_codense_closure(s: Set[Real]) {
    is_nowhere_dense_real_set(s) = is_codense_real_set(closure(s))
} by {
    if is_nowhere_dense_real_set(s) {
        codense_closure_of_nowhere_dense(s)
        is_codense_real_set(closure(s))
    }
    if is_codense_real_set(closure(s)) {
        nowhere_dense_of_codense_closure(s)
        is_nowhere_dense_real_set(s)
    }
    is_nowhere_dense_real_set(s) = is_codense_real_set(closure(s))
}

/// Subsets of closures of nowhere dense sets are nowhere dense.
theorem nowhere_dense_of_subset_closure(s: Set[Real], t: Set[Real]) {
    s.subset(closure(t)) and is_nowhere_dense_real_set(t) implies is_nowhere_dense_real_set(s)
} by {
    if s.subset(closure(t)) and is_nowhere_dense_real_set(t) {
        closure_of_nowhere_dense_real_set_is_nowhere_dense(t)
        is_nowhere_dense_real_set(closure(t))
        nowhere_dense_real_set_of_subset(s, closure(t))
        is_nowhere_dense_real_set(s)
    }
}

/// If a set is contained in a nowhere dense set, its closure has empty interior.
theorem subset_nowhere_dense_imp_empty_interior_closure(s: Set[Real], t: Set[Real]) {
    s.subset(t) and is_nowhere_dense_real_set(t) implies interior(closure(s)) = Set[Real].empty_set
} by {
    if s.subset(t) and is_nowhere_dense_real_set(t) {
        nowhere_dense_real_set_of_subset(s, t)
        is_nowhere_dense_real_set(s)
        interior(closure(s)) = Set[Real].empty_set
    }
}
