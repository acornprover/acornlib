from data.basic.set import Set, compl_contains_eq
from real.real_field import Real
from real.topology import closed_set_contains_adherent, eps_adherent_of_contains_close,
    interior_point_intro, is_adherent_point_of_set, is_closed_set,
    is_eps_adherent_to_set, is_interior_point, is_open_set

/// A point outside a closed set is not adherent to it.
theorem closed_complement_not_adherent(s: Set[Real], x: Real) {
    is_closed_set(s) and s.c.contains(x) implies not is_adherent_point_of_set(s, x)
} by {
    if is_closed_set(s) and s.c.contains(x) {
        compl_contains_eq(s, x)
        not s.contains(x)
        if is_adherent_point_of_set(s, x) {
            closed_set_contains_adherent(s, x)
            false
        }
    }
}

/// Failure of adherence gives a positive epsilon with no set witness.
theorem not_adherent_has_missing_eps(s: Set[Real], x: Real) {
    not is_adherent_point_of_set(s, x) implies exists(eps: Real) {
        eps.is_positive and not is_eps_adherent_to_set(s, x, eps)
    }
} by {
    if not is_adherent_point_of_set(s, x) {
        let eps: Real satisfy {
            eps.is_positive and not is_eps_adherent_to_set(s, x, eps)
        }
        exists(e: Real) {
            e.is_positive and not is_eps_adherent_to_set(s, x, e)
        }
    }
}

/// A missing epsilon-adherence witness excludes close points from the set.
theorem missing_eps_adherent_excludes_member(s: Set[Real], x: Real, eps: Real, y: Real) {
    not is_eps_adherent_to_set(s, x, eps) and y.is_close(x, eps) implies not s.contains(y)
} by {
    if not is_eps_adherent_to_set(s, x, eps) and y.is_close(x, eps) {
        if s.contains(y) {
            eps_adherent_of_contains_close(s, x, y, eps)
            false
        }
    }
}

/// Every point of the complement of a closed set is an interior point of the complement.
theorem closed_complement_interior_point(s: Set[Real], x: Real) {
    is_closed_set(s) and s.c.contains(x) implies is_interior_point(s.c, x)
} by {
    if is_closed_set(s) and s.c.contains(x) {
        closed_complement_not_adherent(s, x)
        not_adherent_has_missing_eps(s, x)
        let eps: Real satisfy {
            eps.is_positive and not is_eps_adherent_to_set(s, x, eps)
        }
        forall(y: Real) {
            if y.is_close(x, eps) {
                missing_eps_adherent_excludes_member(s, x, eps, y)
                not s.contains(y)
                compl_contains_eq(s, y)
                s.c.contains(y)
            }
        }
        interior_point_intro(s.c, x, eps)
        is_interior_point(s.c, x)
    }
}

/// The complement of a closed set is open.
theorem complement_of_closed_is_open(s: Set[Real]) {
    is_closed_set(s) implies is_open_set(s.c)
} by {
    if is_closed_set(s) {
        forall(x: Real) {
            if s.c.contains(x) {
                closed_complement_interior_point(s, x)
            }
        }
        is_open_set(s.c)
    }
}
