from real.continuity_base import Real, continuous_at, continuous_condition
from real.derivative_basic import difference_quotient, has_derivative_at,
    differentiable_at, sub_ne_zero_of_ne
from real.prod_seq import abs_le_abs_add_eps
from real.real_base import abs_gte_zero, abs_not_neg, lte_lt_trans, lt_trans,
    self_close
from real.real_field import mul_div_cancel
from real.real_ring import exists_small_mul_variant_2, lt_mul_pos_left,
    lte_mul_nonneg_right, mul_abs
from real.real_seq import close_and_lt_imp_close, eps_smaller_than_both

/// Division by a nonzero real is canceled by multiplying on the right by the denominator.
theorem div_mul_cancel_denominator(a: Real, b: Real) {
    b != Real.0 implies (a / b) * b = a
} by {
    if b != Real.0 {
        mul_div_cancel(a, b)
        b * (a / b) = a
        (a / b) * b = b * (a / b)
        (a / b) * b = a
    }
}

/// A difference quotient recovers the function increment after multiplication by the input increment.
theorem difference_quotient_mul_increment(f: Real -> Real, x0: Real, x: Real) {
    x != x0 implies difference_quotient(f, x0, x) * (x - x0) = f(x) - f(x0)
} by {
    sub_ne_zero_of_ne(x, x0)
    let increment = x - x0
    let value_increment = f(x) - f(x0)
    increment != Real.0
    difference_quotient(f, x0, x) = value_increment / increment
    div_mul_cancel_denominator(value_increment, increment)
    (value_increment / increment) * increment = value_increment
    difference_quotient(f, x0, x) * increment =
        (value_increment / increment) * increment
    difference_quotient(f, x0, x) * increment = value_increment
    x - x0 = increment
    difference_quotient(f, x0, x) * (x - x0) =
        difference_quotient(f, x0, x) * increment
    difference_quotient(f, x0, x) * (x - x0) = value_increment
}

/// Difference quotients of a differentiable real function are locally bounded near the base point.
theorem derivative_difference_quotient_local_abs_bound(f: Real -> Real, x0: Real, d: Real) {
    has_derivative_at(f, x0, d) implies exists(delta: Real, bound: Real) {
        delta.is_positive and bound.is_positive and forall(x: Real) {
            x != x0 and x.is_close(x0, delta)
            implies difference_quotient(f, x0, x).abs <= bound
        }
    }
} by {
    Real.1.is_positive
    has_derivative_at(f, x0, d) = forall(eps: Real) {
        eps.is_positive implies exists(delta: Real) {
            delta.is_positive and forall(x: Real) {
                x != x0 and x.is_close(x0, delta)
                implies difference_quotient(f, x0, x).is_close(d, eps)
            }
        }
    }
    let delta: Real satisfy {
        delta.is_positive and forall(x: Real) {
            x != x0 and x.is_close(x0, delta)
            implies difference_quotient(f, x0, x).is_close(d, Real.1)
        }
    }
    let bound = d.abs + Real.1
    abs_gte_zero(d)
    d.abs >= Real.0
    d.abs < d.abs + Real.1
    lte_lt_trans(Real.0, d.abs, d.abs + Real.1)
    Real.0 < d.abs + Real.1
    bound > Real.0
    bound.is_positive
    forall(x: Real) {
        if x != x0 and x.is_close(x0, delta) {
            difference_quotient(f, x0, x).is_close(d, Real.1)
            abs_le_abs_add_eps(difference_quotient(f, x0, x), d, Real.1)
            difference_quotient(f, x0, x).abs <= d.abs + Real.1
            difference_quotient(f, x0, x).abs <= bound
        }
    }
    exists(delta2: Real, bound2: Real) {
        delta2.is_positive and bound2.is_positive and forall(x: Real) {
            x != x0 and x.is_close(x0, delta2)
            implies difference_quotient(f, x0, x).abs <= bound2
        }
    }
}

/// A bounded difference quotient turns a small input increment into a small function increment.
theorem bounded_difference_quotient_close(
    f: Real -> Real, x0: Real, x: Real, delta: Real, bound: Real, eps: Real
) {
    x != x0 and x.is_close(x0, delta) and delta.is_positive and bound.is_positive
    and bound * delta < eps
    and difference_quotient(f, x0, x).abs <= bound
    implies f(x).is_close(f(x0), eps)
} by {
    difference_quotient_mul_increment(f, x0, x)
    x != x0
    x.is_close(x0, delta)
    difference_quotient(f, x0, x) * (x - x0) = f(x) - f(x0)
    mul_abs(difference_quotient(f, x0, x), x - x0)
    difference_quotient(f, x0, x).abs * (x - x0).abs =
        (difference_quotient(f, x0, x) * (x - x0)).abs
    (f(x) - f(x0)).abs = difference_quotient(f, x0, x).abs * (x - x0).abs
    x.is_close(x0, delta) = (x - x0).abs < delta
    (x - x0).abs < delta
    abs_not_neg(x - x0)
    not (x - x0).abs.is_negative
    lte_mul_nonneg_right(difference_quotient(f, x0, x).abs, bound, (x - x0).abs)
    difference_quotient(f, x0, x).abs * (x - x0).abs <= bound * (x - x0).abs
    lt_mul_pos_left((x - x0).abs, delta, bound)
    bound.is_positive and (x - x0).abs < delta
    bound * (x - x0).abs < bound * delta
    lte_lt_trans(
        difference_quotient(f, x0, x).abs * (x - x0).abs,
        bound * (x - x0).abs,
        bound * delta
    )
    difference_quotient(f, x0, x).abs * (x - x0).abs < bound * delta
    lt_trans(difference_quotient(f, x0, x).abs * (x - x0).abs, bound * delta, eps)
    difference_quotient(f, x0, x).abs * (x - x0).abs < eps
    (f(x) - f(x0)).abs < eps
    f(x).is_close(f(x0), eps)
}

/// A real function with a derivative at a point is continuous at that point.
theorem derivative_continuous_at(f: Real -> Real, x0: Real, d: Real) {
    has_derivative_at(f, x0, d) implies continuous_at(f, x0)
} by {
    derivative_difference_quotient_local_abs_bound(f, x0, d)
    let (dq_delta: Real, bound: Real) satisfy {
        dq_delta.is_positive and bound.is_positive and forall(x: Real) {
            x != x0 and x.is_close(x0, dq_delta)
            implies difference_quotient(f, x0, x).abs <= bound
        }
    }
    forall(eps: Real) {
        if eps.is_positive {
            exists_small_mul_variant_2(bound, eps)
            let small_delta: Real satisfy {
                small_delta.is_positive and small_delta * bound < eps
            }
            eps_smaller_than_both(dq_delta, small_delta)
            let delta: Real satisfy {
                delta.is_positive and delta < dq_delta and delta < small_delta
            }
            forall(x: Real) {
                if x.is_close(x0, delta) {
                    if x = x0 {
                        self_close(f(x0), eps)
                        f(x).is_close(f(x0), eps)
                    } else {
                        close_and_lt_imp_close(x, x0, delta, dq_delta)
                        close_and_lt_imp_close(x, x0, delta, small_delta)
                        x.is_close(x0, dq_delta)
                        x.is_close(x0, small_delta)
                        difference_quotient(f, x0, x).abs <= bound
                        x.is_close(x0, small_delta) = (x - x0).abs < small_delta
                        (x - x0).abs < small_delta
                        lt_mul_pos_left(delta, small_delta, bound)
                        bound * delta < bound * small_delta
                        bound * small_delta = small_delta * bound
                        bound * delta < eps
                        bounded_difference_quotient_close(f, x0, x, delta, bound, eps)
                        f(x).is_close(f(x0), eps)
                    }
                }
            }
            continuous_condition(f, x0, delta, eps)
            delta.is_positive and continuous_condition(f, x0, delta, eps)
            exists(delta2: Real) {
                delta2.is_positive and continuous_condition(f, x0, delta2, eps)
            }
        }
    }
    continuous_at(f, x0) = forall(eps2: Real) {
        eps2.is_positive implies exists(delta2: Real) {
            delta2.is_positive and continuous_condition(f, x0, delta2, eps2)
        }
    }
    if not continuous_at(f, x0) {
        not forall(eps2: Real) {
            eps2.is_positive implies exists(delta2: Real) {
                delta2.is_positive and continuous_condition(f, x0, delta2, eps2)
            }
        }
        let bad_eps: Real satisfy {
            bad_eps.is_positive and forall(delta2: Real) {
                not (delta2.is_positive and continuous_condition(f, x0, delta2, bad_eps))
            }
        }
        exists(delta2: Real) {
            delta2.is_positive and continuous_condition(f, x0, delta2, bad_eps)
        }
        false
    }
}

/// A real function differentiable at a point is continuous at that point.
theorem differentiable_continuous_at(f: Real -> Real, x0: Real) {
    differentiable_at(f, x0) implies continuous_at(f, x0)
} by {
    let d: Real satisfy {
        has_derivative_at(f, x0, d)
    }
    derivative_continuous_at(f, x0, d)
    continuous_at(f, x0)
}
