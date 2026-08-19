/// Power series analysis: term-by-term differentiation and integration.
///
/// This file formalizes the algebraic core of term-by-term calculus for power
/// series: the coefficient sequence (n + 1) * a(n + 1) obtained by
/// differentiating sum_n a(n) x^n, and the coefficient sequence a(n) / (n + 1)
/// obtained by integrating term by term.  For the exponential, sine and cosine
/// series the differentiated coefficient sequence is exactly the original one
/// (shifted appropriately), so the differentiated series has the same radius
/// of convergence; the integrated coefficient sequence never exceeds the
/// original one, so term-by-term integration preserves absolute convergence.
/// The main results state these identities at the level of the series:
///
///   - the differentiated exponential series is Real.exp (exp_diff_series_eq_exp),
///   - the differentiated sine series is Real.cos and the differentiated cosine
///     series is -Real.sin (sin_diff_series_eq_cos, cos_diff_series_eq_neg_sin),
///   - the integrated exponential series is Real.exp - 1
///     (exp_int_series_eq_exp_sub_one),
///   - the differentiated Mercator series is the alternating geometric series
///     1 / (1 + x) (mercator_diff_series_eq_recip).
///
/// Differentiability of Real.exp, Real.sin and Real.cos from the same series is proved in
/// derivative_exp_log.ac and derivative_trig.ac; the radius-of-convergence
/// lemma for general power series is developed in a parallel worktree
/// (series_deep.ac).

from nat import Nat, one_pow, factorial_step, from_nat, from_nat_one
from rat import Rat
from list import partial, partial_pointwise_eq
from algebra.semigroup import mul_fn
from real.real_field import Real, mul_div_cancel, div_mul_cancel_left, mul_div
from real.real_ring import converges, limit, converges_to, from_nat_is_from_rat, mul_abs, mul_nonneg, mul_le_mul_nonneg
from real.real_seq import converges_imp_converges_to, converges_to_imp_converges, converges_to_unique, tail_bound, tail_bound_implies_is_close
from real.real_series import mul_seq, partial_mul_seq_comm, geom_converges, geom_series_no_div, is_lower_bound, seq_lte, neg_seq, abs_pow, pow_nonneg, converges_mul_seq, mul_seq_converges_to, neg_seq_converges, neg_seq_converges_to
from real.real_base import abs_gte_zero, lte_lt_trans, lt_add_right, abs_neg, lte_abs, lt_lte_trans
from real.abs_conv import absolutely_converges, abs_fn, absolutely_converges_comparison, absolutely_converges_imp_converges
from real.exp import exp_term, exp_term_partial_converges, factorial_pos, factorial_suc_real, abs_div, suc_pos, pow_suc, mul_frac_assoc
from real.derivative_exp_log import exp_shift_term, exp_shift_sum, exp_sub_one_series_sum, exp_shift_term_mul_x, exp_shift_term_abs_converges
from real.trig import sin_term, cos_term, alternating_sign_abs, sin_term_abs_converges, cos_term_abs_converges, real_pow_mul_distrib, mul_neg_one_left
from real.harmonic import harmonic, real_recip_antitone_pos, real_one_div_pos
from real.limit_theorems import limit_one_over_suc, one_over_suc_converges
from real.limits import vanishes
from real.series_tests import alt_term, alternating_series_test, is_decreasing_seq
from real.zeta_values import harmonic_decreasing
from real.integral_exp import from_nat_lte_mono
from algebra.ring.ring import alternating_sign, alternating_sign_suc, alternating_sign_eq_neg_one_pow

numerals Real
numerals Nat

// ---------------------------------------------------------------------------
// Power series in coefficient form.
// ---------------------------------------------------------------------------

/// The nth term of the power series with coefficient sequence a: a(n) * x^n.
define ps_term(a: Nat -> Real, x: Real, n: Nat) -> Real {
    a(n) * x.pow(n)
}

/// The sum of the power series with coefficient sequence a at x, when the series converges.
define ps_sum(a: Nat -> Real, x: Real) -> Real {
    limit(partial(ps_term(a, x)))
}

/// The differentiated coefficient sequence of a power series: (n + 1) * a(n + 1).
/// The power series with these coefficients is the term-by-term derivative of
/// the power series with coefficients a(n).
define ps_diff_coeff(a: Nat -> Real, n: Nat) -> Real {
    Real.from_rat(Rat.from_nat(n.suc)) * a(n.suc)
}

/// The nth term of the differentiated power series: (n + 1) * a(n + 1) * x^n.
define ps_diff_term(a: Nat -> Real, x: Real, n: Nat) -> Real {
    ps_term(ps_diff_coeff(a), x, n)
}

/// The integrated coefficient sequence of a power series: a(n) / (n + 1).
/// The power series with these coefficients is the term-by-term integral of the
/// power series with coefficients a(n), shifted by one power of x.
define ps_int_coeff(a: Nat -> Real, n: Nat) -> Real {
    a(n) / Real.from_rat(Rat.from_nat(n.suc))
}

/// The nth term of the integrated power series: a(n) * x^(n+1) / (n + 1).
define ps_int_term(a: Nat -> Real, x: Real, n: Nat) -> Real {
    a(n) * x.pow(n.suc) / Real.from_rat(Rat.from_nat(n.suc))
}

// ---------------------------------------------------------------------------
// Plumbing for limits of pointwise equal sequences.
// ---------------------------------------------------------------------------

/// Pointwise equal sequences converge to the same limits.
theorem converges_to_pointwise_eq(q: Nat -> Real, r: Nat -> Real, a: Real) {
    (forall(n: Nat) { q(n) = r(n) }) implies (converges_to(q, a) implies converges_to(r, a))
} by {
    if forall(n: Nat) { q(n) = r(n) } {
        if converges_to(q, a) {
            forall(eps: Real) {
                if eps.is_positive {
                    converges_to(q, a) = forall(e: Real) {
                        e.is_positive implies exists(n: Nat) {
                            tail_bound(q, a, n, e)
                        }
                    }
                    eps.is_positive implies exists(n: Nat) {
                        tail_bound(q, a, n, eps)
                    }
                    exists(n: Nat) {
                        tail_bound(q, a, n, eps)
                    }
                    let n: Nat satisfy {
                        tail_bound(q, a, n, eps)
                    }
                    forall(i: Nat) {
                        if n <= i {
                            tail_bound_implies_is_close(q, a, n, eps, i)
                            q(i).is_close(a, eps)
                            (q(i) - a).abs < eps = q(i).is_close(a, eps)
                            (q(i) - a).abs < eps
                            q(i) = r(i)
                            q(i) - a = r(i) - a
                            (r(i) - a).abs < eps
                            (r(i) - a).abs < eps = r(i).is_close(a, eps)
                            r(i).is_close(a, eps)
                        }
                    }
                    exists(k0: Nat) {
                        n <= k0 and not r(k0).is_close(a, eps)
                    } or tail_bound(r, a, n, eps)
                    tail_bound(r, a, n, eps)
                    exists(n0: Nat) {
                        tail_bound(r, a, n0, eps)
                    }
                }
            }
            forall(e: Real) {
                e.is_positive implies exists(n: Nat) {
                    tail_bound(r, a, n, e)
                }
            }
            converges_to(r, a) = forall(e: Real) {
                e.is_positive implies exists(n: Nat) {
                    tail_bound(r, a, n, e)
                }
            }
            converges_to(r, a)
        }
    }
}

/// Pointwise equal sequences converge to the same limits.
theorem converges_pointwise_eq(q: Nat -> Real, r: Nat -> Real) {
    (forall(n: Nat) { q(n) = r(n) }) implies (converges(q) implies converges(r))
} by {
    if forall(n: Nat) { q(n) = r(n) } {
        if converges(q) {
            converges_imp_converges_to(q)
            converges_to(q, limit(q))
            converges_to_pointwise_eq(q, r, limit(q))
            converges_to(r, limit(q))
            converges_to_imp_converges(r, limit(q))
            converges(r)
        }
    }
}

/// Pointwise equal sequences have equal limits, when one of them converges.
theorem limit_pointwise_eq(s: Nat -> Real, t: Nat -> Real) {
    (forall(n: Nat) { s(n) = t(n) }) and converges(s)
    implies
    limit(t) = limit(s)
} by {
    if forall(n: Nat) { s(n) = t(n) } and converges(s) {
        converges_imp_converges_to(s)
        converges_to(s, limit(s))
        converges_to_pointwise_eq(s, t, limit(s))
        converges_to(t, limit(s))
        converges_to_imp_converges(t, limit(s))
        converges(t)
        converges_imp_converges_to(t)
        converges_to(t, limit(t))
        converges_to_unique(t, limit(s), limit(t))
        limit(s) = limit(t)
        limit(t) = limit(s)
    }
}

// ---------------------------------------------------------------------------
// Field identities behind the coefficient algebra.
// ---------------------------------------------------------------------------

/// For nonzero a and b: a * (1 / (a * b)) = 1 / b.
theorem mul_recip_prod_cancel(a: Real, b: Real) {
    a != Real.0 and b != Real.0 implies a * (Real.1 / (a * b)) = Real.1 / b
} by {
    if a != Real.0 and b != Real.0 {
        mul_div(Real.1, a, Real.1, b)
        (Real.1 / a) * (Real.1 / b) = (Real.1 * Real.1) / (a * b)
        Real.1 * Real.1 = Real.1
        (Real.1 / a) * (Real.1 / b) = Real.1 / (a * b)
        Real.1 / (a * b) = (Real.1 / a) * (Real.1 / b)
        a * (Real.1 / (a * b)) = a * ((Real.1 / a) * (Real.1 / b))
        a * ((Real.1 / a) * (Real.1 / b)) = (a * (Real.1 / a)) * (Real.1 / b)
        mul_div_cancel(Real.1, a)
        a * (Real.1 / a) = Real.1
        (a * (Real.1 / a)) * (Real.1 / b) = Real.1 * (Real.1 / b)
        Real.1 * (Real.1 / b) = Real.1 / b
        a * (Real.1 / (a * b)) = Real.1 / b
    }
}

/// For nonzero a and b: a * (c / (a * b)) = c / b.
theorem mul_div_prod_cancel(a: Real, b: Real, c: Real) {
    a != Real.0 and b != Real.0 implies a * (c / (a * b)) = c / b
} by {
    if a != Real.0 and b != Real.0 {
        c / (a * b) = c * (Real.1 / (a * b))
        a * (c / (a * b)) = a * (c * (Real.1 / (a * b)))
        a * (c * (Real.1 / (a * b))) = c * (a * (Real.1 / (a * b)))
        mul_recip_prod_cancel(a, b)
        a * (Real.1 / (a * b)) = Real.1 / b
        c * (a * (Real.1 / (a * b))) = c * (Real.1 / b)
        c * (Real.1 / b) = c / b
        a * (c / (a * b)) = c / b
    }
}

/// Dividing a reciprocal by a nonzero real: (1 / b) / a = 1 / (a * b).
theorem recip_div(a: Real, b: Real) {
    a != Real.0 and b != Real.0 implies (Real.1 / b) / a = Real.1 / (a * b)
} by {
    if a != Real.0 and b != Real.0 {
        mul_div(Real.1, b, Real.1, a)
        (Real.1 / b) * (Real.1 / a) = (Real.1 * Real.1) / (b * a)
        Real.1 * Real.1 = Real.1
        (Real.1 / b) * (Real.1 / a) = Real.1 / (b * a)
        (Real.1 / b) / a = (Real.1 / b) * (Real.1 / a)
        (Real.1 / b) / a = Real.1 / (b * a)
        b * a = a * b
        Real.1 / (b * a) = Real.1 / (a * b)
        (Real.1 / b) / a = Real.1 / (a * b)
    }
}

/// Dividing a nonnegative real by a real at least one does not increase it.
theorem div_nonneg_le_self(c: Real, d: Real) {
    Real.0 <= c and Real.1 <= d implies c / d <= c
} by {
    if Real.0 <= c and Real.1 <= d {
        Real.1 > Real.0
        Real.0 < Real.1
        lt_lte_trans(Real.0, Real.1, d)
        Real.0 < d
        real_recip_antitone_pos(Real.1, d)
        Real.1 / d <= Real.1 / Real.1
        Real.1 / Real.1 = Real.1
        Real.1 / d <= Real.1
        real_one_div_pos(d)
        Real.1 / d > Real.0
        Real.0 <= Real.1 / d
        c / d = c * (Real.1 / d)
        mul_le_mul_nonneg(c, Real.1 / d, c, Real.1)
        c * (Real.1 / d) <= c * Real.1
        c * Real.1 = c
        c / d <= c
    }
}

// ---------------------------------------------------------------------------
// The exponential series.
// ---------------------------------------------------------------------------

/// The coefficient sequence of the exponential series: 1 / n!.
define exp_coeff(n: Nat) -> Real {
    Real.1 / Real.from_rat(Rat.from_nat(n.factorial))
}

/// The nth term of the exponential power series in coefficient form is the nth
/// exponential term: (1/n!) x^n = x^n / n!.
theorem ps_term_exp_coeff(x: Real, n: Nat) {
    ps_term(exp_coeff, x, n) = exp_term(x, n)
} by {
    ps_term(exp_coeff, x, n) = exp_coeff(n) * x.pow(n)
    exp_coeff(n) = Real.1 / Real.from_rat(Rat.from_nat(n.factorial))
    ps_term(exp_coeff, x, n) = (Real.1 / Real.from_rat(Rat.from_nat(n.factorial))) * x.pow(n)
    (Real.1 / Real.from_rat(Rat.from_nat(n.factorial))) * x.pow(n) =
        x.pow(n) / Real.from_rat(Rat.from_nat(n.factorial))
    exp_term(x, n) = x.pow(n) / Real.from_rat(Rat.from_nat(n.factorial))
    ps_term(exp_coeff, x, n) = exp_term(x, n)
}

/// Differentiating the exponential coefficients does not change them:
/// (n + 1) * (1 / (n + 1)!) = 1 / n!.
theorem exp_diff_coeff_eq_exp_coeff(n: Nat) {
    ps_diff_coeff(exp_coeff, n) = exp_coeff(n)
} by {
    ps_diff_coeff(exp_coeff, n) = Real.from_rat(Rat.from_nat(n.suc)) * exp_coeff(n.suc)
    exp_coeff(n.suc) = Real.1 / Real.from_rat(Rat.from_nat(n.suc.factorial))
    ps_diff_coeff(exp_coeff, n) =
        Real.from_rat(Rat.from_nat(n.suc)) * (Real.1 / Real.from_rat(Rat.from_nat(n.suc.factorial)))
    factorial_suc_real(n)
    Real.from_rat(Rat.from_nat(n.suc.factorial)) =
        Real.from_rat(Rat.from_nat(n.suc)) * Real.from_rat(Rat.from_nat(n.factorial))
    Real.1 / Real.from_rat(Rat.from_nat(n.suc.factorial)) =
        Real.1 / (Real.from_rat(Rat.from_nat(n.suc)) * Real.from_rat(Rat.from_nat(n.factorial)))
    suc_pos(n)
    Real.from_rat(Rat.from_nat(n.suc)) > Real.0
    Real.from_rat(Rat.from_nat(n.suc)) != Real.0
    factorial_pos(n)
    Real.from_rat(Rat.from_nat(n.factorial)) > Real.0
    Real.from_rat(Rat.from_nat(n.factorial)) != Real.0
    mul_recip_prod_cancel(Real.from_rat(Rat.from_nat(n.suc)), Real.from_rat(Rat.from_nat(n.factorial)))
    Real.from_rat(Rat.from_nat(n.suc)) *
        (Real.1 / (Real.from_rat(Rat.from_nat(n.suc)) * Real.from_rat(Rat.from_nat(n.factorial)))) =
        Real.1 / Real.from_rat(Rat.from_nat(n.factorial))
    Real.from_rat(Rat.from_nat(n.suc)) *
        (Real.1 / Real.from_rat(Rat.from_nat(n.suc.factorial))) =
        Real.from_rat(Rat.from_nat(n.suc)) *
        (Real.1 / (Real.from_rat(Rat.from_nat(n.suc)) * Real.from_rat(Rat.from_nat(n.factorial))))
    ps_diff_coeff(exp_coeff, n) = Real.1 / Real.from_rat(Rat.from_nat(n.factorial))
    exp_coeff(n) = Real.1 / Real.from_rat(Rat.from_nat(n.factorial))
    ps_diff_coeff(exp_coeff, n) = exp_coeff(n)
}

/// Integrating the exponential coefficients shifts them: (1/n!) / (n + 1) = 1 / (n + 1)!.
theorem exp_int_coeff_eq_shifted(n: Nat) {
    ps_int_coeff(exp_coeff, n) = exp_coeff(n.suc)
} by {
    ps_int_coeff(exp_coeff, n) = exp_coeff(n) / Real.from_rat(Rat.from_nat(n.suc))
    exp_coeff(n) = Real.1 / Real.from_rat(Rat.from_nat(n.factorial))
    ps_int_coeff(exp_coeff, n) =
        (Real.1 / Real.from_rat(Rat.from_nat(n.factorial))) / Real.from_rat(Rat.from_nat(n.suc))
    suc_pos(n)
    Real.from_rat(Rat.from_nat(n.suc)) != Real.0
    factorial_pos(n)
    Real.from_rat(Rat.from_nat(n.factorial)) != Real.0
    recip_div(Real.from_rat(Rat.from_nat(n.suc)), Real.from_rat(Rat.from_nat(n.factorial)))
    (Real.1 / Real.from_rat(Rat.from_nat(n.factorial))) / Real.from_rat(Rat.from_nat(n.suc)) =
        Real.1 / (Real.from_rat(Rat.from_nat(n.suc)) * Real.from_rat(Rat.from_nat(n.factorial)))
    factorial_suc_real(n)
    Real.from_rat(Rat.from_nat(n.suc.factorial)) =
        Real.from_rat(Rat.from_nat(n.suc)) * Real.from_rat(Rat.from_nat(n.factorial))
    Real.1 / (Real.from_rat(Rat.from_nat(n.suc)) * Real.from_rat(Rat.from_nat(n.factorial))) =
        Real.1 / Real.from_rat(Rat.from_nat(n.suc.factorial))
    ps_int_coeff(exp_coeff, n) = Real.1 / Real.from_rat(Rat.from_nat(n.suc.factorial))
    exp_coeff(n.suc) = Real.1 / Real.from_rat(Rat.from_nat(n.suc.factorial))
    ps_int_coeff(exp_coeff, n) = exp_coeff(n.suc)
}

/// The differentiated exponential terms are exactly the exponential terms:
/// (n + 1) x^n / (n + 1)! = x^n / n!.
theorem ps_diff_term_exp_coeff_eq_exp_term(x: Real, n: Nat) {
    ps_diff_term(exp_coeff, x, n) = exp_term(x, n)
} by {
    ps_diff_term(exp_coeff, x, n) = ps_term(ps_diff_coeff(exp_coeff), x, n)
    ps_term(ps_diff_coeff(exp_coeff), x, n) = ps_diff_coeff(exp_coeff, n) * x.pow(n)
    exp_diff_coeff_eq_exp_coeff(n)
    ps_diff_coeff(exp_coeff, n) = exp_coeff(n)
    ps_diff_coeff(exp_coeff, n) * x.pow(n) = exp_coeff(n) * x.pow(n)
    ps_term(exp_coeff, x, n) = exp_coeff(n) * x.pow(n)
    ps_diff_term(exp_coeff, x, n) = ps_term(exp_coeff, x, n)
    ps_term_exp_coeff(x, n)
    ps_term(exp_coeff, x, n) = exp_term(x, n)
    ps_diff_term(exp_coeff, x, n) = exp_term(x, n)
}

/// The integrated exponential terms are the exponential terms shifted by one index:
/// (1/n!) x^(n+1) / (n + 1) = x^(n+1) / (n + 1)!.
theorem ps_int_term_exp_coeff_eq_shift(x: Real, n: Nat) {
    ps_int_term(exp_coeff, x, n) = exp_term(x, n.suc)
} by {
    ps_int_term(exp_coeff, x, n) = exp_coeff(n) * x.pow(n.suc) / Real.from_rat(Rat.from_nat(n.suc))
    mul_frac_assoc(exp_coeff(n), x.pow(n.suc), Real.from_rat(Rat.from_nat(n.suc)))
    exp_coeff(n) * (x.pow(n.suc) / Real.from_rat(Rat.from_nat(n.suc))) =
        exp_coeff(n) * x.pow(n.suc) / Real.from_rat(Rat.from_nat(n.suc))
    exp_coeff(n) * x.pow(n.suc) / Real.from_rat(Rat.from_nat(n.suc)) =
        exp_coeff(n) * (x.pow(n.suc) / Real.from_rat(Rat.from_nat(n.suc)))
    exp_coeff(n) * (x.pow(n.suc) / Real.from_rat(Rat.from_nat(n.suc))) =
        (exp_coeff(n) / Real.from_rat(Rat.from_nat(n.suc))) * x.pow(n.suc)
    exp_coeff(n) * x.pow(n.suc) / Real.from_rat(Rat.from_nat(n.suc)) =
        (exp_coeff(n) / Real.from_rat(Rat.from_nat(n.suc))) * x.pow(n.suc)
    ps_int_coeff(exp_coeff, n) = exp_coeff(n) / Real.from_rat(Rat.from_nat(n.suc))
    exp_coeff(n) / Real.from_rat(Rat.from_nat(n.suc)) = ps_int_coeff(exp_coeff, n)
    exp_int_coeff_eq_shifted(n)
    ps_int_coeff(exp_coeff, n) = exp_coeff(n.suc)
    exp_coeff(n) / Real.from_rat(Rat.from_nat(n.suc)) = exp_coeff(n.suc)
    (exp_coeff(n) / Real.from_rat(Rat.from_nat(n.suc))) * x.pow(n.suc) =
        exp_coeff(n.suc) * x.pow(n.suc)
    ps_int_term(exp_coeff, x, n) = exp_coeff(n.suc) * x.pow(n.suc)
    ps_term(exp_coeff, x, n.suc) = exp_coeff(n.suc) * x.pow(n.suc)
    ps_term_exp_coeff(x, n.suc)
    ps_term(exp_coeff, x, n.suc) = exp_term(x, n.suc)
    exp_coeff(n.suc) * x.pow(n.suc) = exp_term(x, n.suc)
    ps_int_term(exp_coeff, x, n) = exp_term(x, n.suc)
}

/// The exponential series is invariant under term-by-term differentiation:
/// sum_{n>=0} (n + 1) x^n / (n + 1)! = x.exp.
theorem exp_diff_series_eq_exp(x: Real) {
    limit(partial(ps_diff_term(exp_coeff, x))) = x.exp
} by {
    forall(n: Nat) {
        forall(k: Nat) {
            if k < n {
                ps_diff_term_exp_coeff_eq_exp_term(x, k)
                ps_diff_term(exp_coeff, x, k) = exp_term(x, k)
            }
        }
        partial_pointwise_eq(ps_diff_term(exp_coeff, x), exp_term(x), n)
        partial(ps_diff_term(exp_coeff, x), n) = partial(exp_term(x), n)
        partial(exp_term(x), n) = partial(ps_diff_term(exp_coeff, x), n)
    }
    exp_term_partial_converges(x)
    converges(partial(exp_term(x)))
    limit_pointwise_eq(partial(exp_term(x)), partial(ps_diff_term(exp_coeff, x)))
    limit(partial(ps_diff_term(exp_coeff, x))) = limit(partial(exp_term(x)))
    x.exp = limit(partial(exp_term(x)))
    limit(partial(ps_diff_term(exp_coeff, x))) = x.exp
}

/// The integrated exponential series sums to x.exp - 1:
/// sum_{n>=0} x^(n+1) / (n + 1)! = x.exp - 1.
theorem exp_int_series_eq_exp_sub_one(x: Real) {
    limit(partial(ps_int_term(exp_coeff, x))) = x.exp - Real.1
} by {
    forall(n: Nat) {
        forall(k: Nat) {
            if k < n {
                ps_int_term_exp_coeff_eq_shift(x, k)
                ps_int_term(exp_coeff, x, k) = exp_term(x, k.suc)
                exp_shift_term_mul_x(x, k)
                x * exp_shift_term(x, k) = exp_term(x, k.suc)
                exp_term(x, k.suc) = x * exp_shift_term(x, k)
                ps_int_term(exp_coeff, x, k) = x * exp_shift_term(x, k)
            }
        }
        partial_pointwise_eq(ps_int_term(exp_coeff, x), mul_seq(x, exp_shift_term(x)), n)
        partial(ps_int_term(exp_coeff, x), n) = partial(mul_seq(x, exp_shift_term(x)), n)
        partial(mul_seq(x, exp_shift_term(x)), n) = partial(ps_int_term(exp_coeff, x), n)
    }
    exp_shift_term_abs_converges(x)
    absolutely_converges(exp_shift_term(x))
    absolutely_converges_imp_converges(exp_shift_term(x))
    converges(partial(exp_shift_term(x)))
    converges_mul_seq(x, partial(exp_shift_term(x)))
    converges(mul_seq(x, partial(exp_shift_term(x))))
    forall(n: Nat) {
        partial_mul_seq_comm(x, exp_shift_term(x))
        partial(mul_seq(x, exp_shift_term(x)), n) = mul_seq(x, partial(exp_shift_term(x)), n)
        mul_seq(x, partial(exp_shift_term(x)), n) = partial(mul_seq(x, exp_shift_term(x)), n)
    }
    limit_pointwise_eq(mul_seq(x, partial(exp_shift_term(x))), partial(mul_seq(x, exp_shift_term(x))))
    limit(partial(mul_seq(x, exp_shift_term(x)))) = limit(mul_seq(x, partial(exp_shift_term(x))))
    converges_pointwise_eq(mul_seq(x, partial(exp_shift_term(x))), partial(mul_seq(x, exp_shift_term(x))))
    converges(partial(mul_seq(x, exp_shift_term(x))))
    limit_pointwise_eq(partial(mul_seq(x, exp_shift_term(x))), partial(ps_int_term(exp_coeff, x)))
    limit(partial(ps_int_term(exp_coeff, x))) = limit(partial(mul_seq(x, exp_shift_term(x))))
    limit(partial(ps_int_term(exp_coeff, x))) = limit(mul_seq(x, partial(exp_shift_term(x))))
    mul_seq_converges_to(x, partial(exp_shift_term(x)))
    converges_to(mul_seq(x, partial(exp_shift_term(x))), x * limit(partial(exp_shift_term(x))))
    converges_imp_converges_to(mul_seq(x, partial(exp_shift_term(x))))
    converges_to(mul_seq(x, partial(exp_shift_term(x))), limit(mul_seq(x, partial(exp_shift_term(x)))))
    converges_to_unique(mul_seq(x, partial(exp_shift_term(x))),
        x * limit(partial(exp_shift_term(x))), limit(mul_seq(x, partial(exp_shift_term(x)))))
    x * limit(partial(exp_shift_term(x))) = limit(mul_seq(x, partial(exp_shift_term(x))))
    limit(mul_seq(x, partial(exp_shift_term(x)))) = x * limit(partial(exp_shift_term(x)))
    limit(partial(ps_int_term(exp_coeff, x))) = x * limit(partial(exp_shift_term(x)))
    exp_shift_sum(x) = limit(partial(exp_shift_term(x)))
    limit(partial(ps_int_term(exp_coeff, x))) = x * exp_shift_sum(x)
    exp_sub_one_series_sum(x)
    x.exp - Real.1 = x * exp_shift_sum(x)
    limit(partial(ps_int_term(exp_coeff, x))) = x.exp - Real.1
}

// ---------------------------------------------------------------------------
// The sine and cosine series.
// ---------------------------------------------------------------------------

/// The nth term of the differentiated sine series: (2n + 1)(-1)^n x^(2n) / (2n + 1)!.
/// This is the derivative of the nth sine term.
define sin_diff_term(x: Real, n: Nat) -> Real {
    Real.from_rat(Rat.from_nat(Nat.2 * n + Nat.1)) * alternating_sign[Real](n) *
        x.pow(Nat.2 * n) / Real.from_rat(Rat.from_nat((Nat.2 * n + Nat.1).factorial))
}

/// Differentiating the sine terms gives the cosine terms:
/// (2n + 1)(-1)^n x^(2n) / (2n + 1)! = (-1)^n x^(2n) / (2n)!.
theorem sin_diff_term_eq_cos_term(x: Real, n: Nat) {
    sin_diff_term(x, n) = cos_term(x, n)
} by {
    sin_diff_term(x, n) = Real.from_rat(Rat.from_nat(Nat.2 * n + Nat.1)) * alternating_sign[Real](n) *
        x.pow(Nat.2 * n) / Real.from_rat(Rat.from_nat((Nat.2 * n + Nat.1).factorial))
    Real.from_rat(Rat.from_nat(Nat.2 * n + Nat.1)) * alternating_sign[Real](n) * x.pow(Nat.2 * n) =
        Real.from_rat(Rat.from_nat(Nat.2 * n + Nat.1)) * (alternating_sign[Real](n) * x.pow(Nat.2 * n))
    sin_diff_term(x, n) = Real.from_rat(Rat.from_nat(Nat.2 * n + Nat.1)) *
        (alternating_sign[Real](n) * x.pow(Nat.2 * n)) /
        Real.from_rat(Rat.from_nat((Nat.2 * n + Nat.1).factorial))
    Nat.2 * n + Nat.1 = (Nat.2 * n).suc
    (Nat.2 * n + Nat.1).factorial = (Nat.2 * n).suc.factorial
    factorial_step(Nat.2 * n)
    (Nat.2 * n).suc.factorial = (Nat.2 * n).suc * (Nat.2 * n).factorial
    (Nat.2 * n + Nat.1).factorial = (Nat.2 * n + Nat.1) * (Nat.2 * n).factorial
    factorial_suc_real(Nat.2 * n)
    Real.from_rat(Rat.from_nat((Nat.2 * n).suc.factorial)) =
        Real.from_rat(Rat.from_nat((Nat.2 * n).suc)) * Real.from_rat(Rat.from_nat((Nat.2 * n).factorial))
    Real.from_rat(Rat.from_nat((Nat.2 * n + Nat.1).factorial)) =
        Real.from_rat(Rat.from_nat(Nat.2 * n + Nat.1)) * Real.from_rat(Rat.from_nat((Nat.2 * n).factorial))
    Real.from_rat(Rat.from_nat(Nat.2 * n + Nat.1)) *
        (alternating_sign[Real](n) * x.pow(Nat.2 * n)) /
        Real.from_rat(Rat.from_nat((Nat.2 * n + Nat.1).factorial)) =
        Real.from_rat(Rat.from_nat(Nat.2 * n + Nat.1)) *
        (alternating_sign[Real](n) * x.pow(Nat.2 * n)) /
        (Real.from_rat(Rat.from_nat(Nat.2 * n + Nat.1)) * Real.from_rat(Rat.from_nat((Nat.2 * n).factorial)))
    mul_frac_assoc(Real.from_rat(Rat.from_nat(Nat.2 * n + Nat.1)),
        alternating_sign[Real](n) * x.pow(Nat.2 * n),
        Real.from_rat(Rat.from_nat(Nat.2 * n + Nat.1)) * Real.from_rat(Rat.from_nat((Nat.2 * n).factorial)))
    Real.from_rat(Rat.from_nat(Nat.2 * n + Nat.1)) *
        ((alternating_sign[Real](n) * x.pow(Nat.2 * n)) /
        (Real.from_rat(Rat.from_nat(Nat.2 * n + Nat.1)) * Real.from_rat(Rat.from_nat((Nat.2 * n).factorial)))) =
        Real.from_rat(Rat.from_nat(Nat.2 * n + Nat.1)) *
        (alternating_sign[Real](n) * x.pow(Nat.2 * n)) /
        (Real.from_rat(Rat.from_nat(Nat.2 * n + Nat.1)) * Real.from_rat(Rat.from_nat((Nat.2 * n).factorial)))
    suc_pos(Nat.2 * n)
    Real.from_rat(Rat.from_nat((Nat.2 * n).suc)) > Real.0
    Real.from_rat(Rat.from_nat(Nat.2 * n + Nat.1)) > Real.0
    Real.from_rat(Rat.from_nat(Nat.2 * n + Nat.1)) != Real.0
    factorial_pos(Nat.2 * n)
    Real.from_rat(Rat.from_nat((Nat.2 * n).factorial)) > Real.0
    Real.from_rat(Rat.from_nat((Nat.2 * n).factorial)) != Real.0
    mul_div_prod_cancel(Real.from_rat(Rat.from_nat(Nat.2 * n + Nat.1)),
        Real.from_rat(Rat.from_nat((Nat.2 * n).factorial)),
        alternating_sign[Real](n) * x.pow(Nat.2 * n))
    Real.from_rat(Rat.from_nat(Nat.2 * n + Nat.1)) *
        ((alternating_sign[Real](n) * x.pow(Nat.2 * n)) /
        (Real.from_rat(Rat.from_nat(Nat.2 * n + Nat.1)) * Real.from_rat(Rat.from_nat((Nat.2 * n).factorial)))) =
        (alternating_sign[Real](n) * x.pow(Nat.2 * n)) / Real.from_rat(Rat.from_nat((Nat.2 * n).factorial))
    sin_diff_term(x, n) =
        (alternating_sign[Real](n) * x.pow(Nat.2 * n)) / Real.from_rat(Rat.from_nat((Nat.2 * n).factorial))
    cos_term(x, n) = alternating_sign[Real](n) * x.pow(Nat.2 * n) / Real.from_rat(Rat.from_nat((Nat.2 * n).factorial))
    (alternating_sign[Real](n) * x.pow(Nat.2 * n)) / Real.from_rat(Rat.from_nat((Nat.2 * n).factorial)) =
        cos_term(x, n)
    sin_diff_term(x, n) = cos_term(x, n)
}

/// The nth term of the differentiated cosine series: the derivative of the
/// (n + 1)-st cosine term: (2n + 2)(-1)^(n+1) x^(2n+1) / (2n + 2)!.
define cos_diff_term(x: Real, n: Nat) -> Real {
    Real.from_rat(Rat.from_nat(Nat.2 * n + Nat.2)) * alternating_sign[Real](n.suc) *
        x.pow(Nat.2 * n + Nat.1) / Real.from_rat(Rat.from_nat((Nat.2 * n + Nat.2).factorial))
}

/// Differentiating the cosine terms gives the negated sine terms:
/// (2n + 2)(-1)^(n+1) x^(2n+1) / (2n + 2)! = -(-1)^n x^(2n+1) / (2n + 1)!.
theorem cos_diff_term_eq_neg_sin_term(x: Real, n: Nat) {
    cos_diff_term(x, n) = -sin_term(x, n)
} by {
    cos_diff_term(x, n) = Real.from_rat(Rat.from_nat(Nat.2 * n + Nat.2)) * alternating_sign[Real](n.suc) *
        x.pow(Nat.2 * n + Nat.1) / Real.from_rat(Rat.from_nat((Nat.2 * n + Nat.2).factorial))
    Real.from_rat(Rat.from_nat(Nat.2 * n + Nat.2)) * alternating_sign[Real](n.suc) * x.pow(Nat.2 * n + Nat.1) =
        Real.from_rat(Rat.from_nat(Nat.2 * n + Nat.2)) *
        (alternating_sign[Real](n.suc) * x.pow(Nat.2 * n + Nat.1))
    cos_diff_term(x, n) = Real.from_rat(Rat.from_nat(Nat.2 * n + Nat.2)) *
        (alternating_sign[Real](n.suc) * x.pow(Nat.2 * n + Nat.1)) /
        Real.from_rat(Rat.from_nat((Nat.2 * n + Nat.2).factorial))
    Nat.2 * n + Nat.2 = (Nat.2 * n + Nat.1).suc
    (Nat.2 * n + Nat.2).factorial = (Nat.2 * n + Nat.1).suc.factorial
    factorial_step(Nat.2 * n + Nat.1)
    (Nat.2 * n + Nat.1).suc.factorial = (Nat.2 * n + Nat.1).suc * (Nat.2 * n + Nat.1).factorial
    (Nat.2 * n + Nat.2).factorial = (Nat.2 * n + Nat.2) * (Nat.2 * n + Nat.1).factorial
    factorial_suc_real(Nat.2 * n + Nat.1)
    Real.from_rat(Rat.from_nat((Nat.2 * n + Nat.1).suc.factorial)) =
        Real.from_rat(Rat.from_nat((Nat.2 * n + Nat.1).suc)) * Real.from_rat(Rat.from_nat((Nat.2 * n + Nat.1).factorial))
    Real.from_rat(Rat.from_nat((Nat.2 * n + Nat.2).factorial)) =
        Real.from_rat(Rat.from_nat(Nat.2 * n + Nat.2)) * Real.from_rat(Rat.from_nat((Nat.2 * n + Nat.1).factorial))
    Real.from_rat(Rat.from_nat(Nat.2 * n + Nat.2)) *
        (alternating_sign[Real](n.suc) * x.pow(Nat.2 * n + Nat.1)) /
        Real.from_rat(Rat.from_nat((Nat.2 * n + Nat.2).factorial)) =
        Real.from_rat(Rat.from_nat(Nat.2 * n + Nat.2)) *
        (alternating_sign[Real](n.suc) * x.pow(Nat.2 * n + Nat.1)) /
        (Real.from_rat(Rat.from_nat(Nat.2 * n + Nat.2)) * Real.from_rat(Rat.from_nat((Nat.2 * n + Nat.1).factorial)))
    mul_frac_assoc(Real.from_rat(Rat.from_nat(Nat.2 * n + Nat.2)),
        alternating_sign[Real](n.suc) * x.pow(Nat.2 * n + Nat.1),
        Real.from_rat(Rat.from_nat(Nat.2 * n + Nat.2)) * Real.from_rat(Rat.from_nat((Nat.2 * n + Nat.1).factorial)))
    Real.from_rat(Rat.from_nat(Nat.2 * n + Nat.2)) *
        ((alternating_sign[Real](n.suc) * x.pow(Nat.2 * n + Nat.1)) /
        (Real.from_rat(Rat.from_nat(Nat.2 * n + Nat.2)) * Real.from_rat(Rat.from_nat((Nat.2 * n + Nat.1).factorial)))) =
        Real.from_rat(Rat.from_nat(Nat.2 * n + Nat.2)) *
        (alternating_sign[Real](n.suc) * x.pow(Nat.2 * n + Nat.1)) /
        (Real.from_rat(Rat.from_nat(Nat.2 * n + Nat.2)) * Real.from_rat(Rat.from_nat((Nat.2 * n + Nat.1).factorial)))
    suc_pos(Nat.2 * n + Nat.1)
    Real.from_rat(Rat.from_nat((Nat.2 * n + Nat.1).suc)) > Real.0
    Real.from_rat(Rat.from_nat(Nat.2 * n + Nat.2)) > Real.0
    Real.from_rat(Rat.from_nat(Nat.2 * n + Nat.2)) != Real.0
    factorial_pos(Nat.2 * n + Nat.1)
    Real.from_rat(Rat.from_nat((Nat.2 * n + Nat.1).factorial)) > Real.0
    Real.from_rat(Rat.from_nat((Nat.2 * n + Nat.1).factorial)) != Real.0
    mul_div_prod_cancel(Real.from_rat(Rat.from_nat(Nat.2 * n + Nat.2)),
        Real.from_rat(Rat.from_nat((Nat.2 * n + Nat.1).factorial)),
        alternating_sign[Real](n.suc) * x.pow(Nat.2 * n + Nat.1))
    Real.from_rat(Rat.from_nat(Nat.2 * n + Nat.2)) *
        ((alternating_sign[Real](n.suc) * x.pow(Nat.2 * n + Nat.1)) /
        (Real.from_rat(Rat.from_nat(Nat.2 * n + Nat.2)) * Real.from_rat(Rat.from_nat((Nat.2 * n + Nat.1).factorial)))) =
        (alternating_sign[Real](n.suc) * x.pow(Nat.2 * n + Nat.1)) / Real.from_rat(Rat.from_nat((Nat.2 * n + Nat.1).factorial))
    cos_diff_term(x, n) =
        (alternating_sign[Real](n.suc) * x.pow(Nat.2 * n + Nat.1)) / Real.from_rat(Rat.from_nat((Nat.2 * n + Nat.1).factorial))
    alternating_sign_suc[Real](n)
    alternating_sign[Real](n.suc) = -alternating_sign[Real](n)
    (alternating_sign[Real](n.suc) * x.pow(Nat.2 * n + Nat.1)) / Real.from_rat(Rat.from_nat((Nat.2 * n + Nat.1).factorial)) =
        (-alternating_sign[Real](n) * x.pow(Nat.2 * n + Nat.1)) / Real.from_rat(Rat.from_nat((Nat.2 * n + Nat.1).factorial))
    -alternating_sign[Real](n) * x.pow(Nat.2 * n + Nat.1) = -(alternating_sign[Real](n) * x.pow(Nat.2 * n + Nat.1))
    (-alternating_sign[Real](n) * x.pow(Nat.2 * n + Nat.1)) / Real.from_rat(Rat.from_nat((Nat.2 * n + Nat.1).factorial)) =
        -(alternating_sign[Real](n) * x.pow(Nat.2 * n + Nat.1)) / Real.from_rat(Rat.from_nat((Nat.2 * n + Nat.1).factorial))
    -(alternating_sign[Real](n) * x.pow(Nat.2 * n + Nat.1)) / Real.from_rat(Rat.from_nat((Nat.2 * n + Nat.1).factorial)) =
        -((alternating_sign[Real](n) * x.pow(Nat.2 * n + Nat.1)) / Real.from_rat(Rat.from_nat((Nat.2 * n + Nat.1).factorial)))
    cos_diff_term(x, n) =
        -((alternating_sign[Real](n) * x.pow(Nat.2 * n + Nat.1)) / Real.from_rat(Rat.from_nat((Nat.2 * n + Nat.1).factorial)))
    sin_term(x, n) = alternating_sign[Real](n) * x.pow(Nat.2 * n + Nat.1) / Real.from_rat(Rat.from_nat((Nat.2 * n + Nat.1).factorial))
    cos_diff_term(x, n) = -sin_term(x, n)
}

/// The sine series differentiates term by term to the cosine series:
/// sum_{n>=0} (2n + 1)(-1)^n x^(2n) / (2n + 1)! = x.cos.
theorem sin_diff_series_eq_cos(x: Real) {
    limit(partial(sin_diff_term(x))) = x.cos
} by {
    forall(n: Nat) {
        forall(k: Nat) {
            if k < n {
                sin_diff_term_eq_cos_term(x, k)
                sin_diff_term(x, k) = cos_term(x, k)
            }
        }
        partial_pointwise_eq(sin_diff_term(x), cos_term(x), n)
        partial(sin_diff_term(x), n) = partial(cos_term(x), n)
        partial(cos_term(x), n) = partial(sin_diff_term(x), n)
    }
    cos_term_abs_converges(x)
    absolutely_converges(cos_term(x))
    absolutely_converges_imp_converges(cos_term(x))
    converges(partial(cos_term(x)))
    limit_pointwise_eq(partial(cos_term(x)), partial(sin_diff_term(x)))
    limit(partial(sin_diff_term(x))) = limit(partial(cos_term(x)))
    x.cos = limit(partial(cos_term(x)))
    limit(partial(sin_diff_term(x))) = x.cos
}

/// The cosine series differentiates term by term to the negated sine series:
/// sum_{n>=0} (2n + 2)(-1)^(n+1) x^(2n+1) / (2n + 2)! = -x.sin.
theorem cos_diff_series_eq_neg_sin(x: Real) {
    limit(partial(cos_diff_term(x))) = -x.sin
} by {
    forall(n: Nat) {
        forall(k: Nat) {
            if k < n {
                cos_diff_term_eq_neg_sin_term(x, k)
                cos_diff_term(x, k) = -sin_term(x, k)
                neg_seq(sin_term(x), k) = -sin_term(x, k)
                cos_diff_term(x, k) = neg_seq(sin_term(x), k)
            }
        }
        partial_pointwise_eq(cos_diff_term(x), neg_seq(sin_term(x)), n)
        partial(cos_diff_term(x), n) = partial(neg_seq(sin_term(x)), n)
        partial(neg_seq(sin_term(x)), n) = partial(cos_diff_term(x), n)
    }
    sin_term_abs_converges(x)
    absolutely_converges(sin_term(x))
    absolutely_converges_imp_converges(sin_term(x))
    converges(partial(sin_term(x)))
    neg_seq_converges(partial(sin_term(x)))
    converges(neg_seq(partial(sin_term(x))))
    forall(n: Nat) {
        partial_mul_seq_comm(-Real.1, sin_term(x))
        partial(mul_seq(-Real.1, sin_term(x)), n) = mul_seq(-Real.1, partial(sin_term(x)), n)
        neg_seq(sin_term(x)) = mul_seq(-Real.1, sin_term(x))
        partial(neg_seq(sin_term(x)), n) = partial(mul_seq(-Real.1, sin_term(x)), n)
        partial(neg_seq(sin_term(x)), n) = mul_seq(-Real.1, partial(sin_term(x)), n)
        mul_seq(-Real.1, partial(sin_term(x)), n) = neg_seq(partial(sin_term(x)), n)
        partial(neg_seq(sin_term(x)), n) = neg_seq(partial(sin_term(x)), n)
        neg_seq(partial(sin_term(x)), n) = partial(neg_seq(sin_term(x)), n)
    }
    limit_pointwise_eq(neg_seq(partial(sin_term(x))), partial(neg_seq(sin_term(x))))
    limit(partial(neg_seq(sin_term(x)))) = limit(neg_seq(partial(sin_term(x))))
    neg_seq_converges_to(partial(sin_term(x)))
    converges_to(neg_seq(partial(sin_term(x))), -limit(partial(sin_term(x))))
    converges_imp_converges_to(neg_seq(partial(sin_term(x))))
    converges_to(neg_seq(partial(sin_term(x))), limit(neg_seq(partial(sin_term(x)))))
    converges_to_unique(neg_seq(partial(sin_term(x))), -limit(partial(sin_term(x))),
        limit(neg_seq(partial(sin_term(x)))))
    -limit(partial(sin_term(x))) = limit(neg_seq(partial(sin_term(x))))
    limit(neg_seq(partial(sin_term(x)))) = -limit(partial(sin_term(x)))
    limit(partial(neg_seq(sin_term(x)))) = -limit(partial(sin_term(x)))
    x.sin = limit(partial(sin_term(x)))
    limit(partial(neg_seq(sin_term(x)))) = -x.sin
    converges_pointwise_eq(neg_seq(partial(sin_term(x))), partial(neg_seq(sin_term(x))))
    converges(partial(neg_seq(sin_term(x))))
    limit_pointwise_eq(partial(neg_seq(sin_term(x))), partial(cos_diff_term(x)))
    limit(partial(cos_diff_term(x))) = limit(partial(neg_seq(sin_term(x))))
    limit(partial(cos_diff_term(x))) = -x.sin
}

// ---------------------------------------------------------------------------
// Term-by-term integration preserves absolute convergence.
// ---------------------------------------------------------------------------

/// Integrating coefficients does not increase them: |a(n)| / (n + 1) <= |a(n)|.
theorem ps_int_coeff_abs_le(a: Nat -> Real, n: Nat) {
    ps_int_coeff(a, n).abs <= a(n).abs
} by {
    ps_int_coeff(a, n) = a(n) / Real.from_rat(Rat.from_nat(n.suc))
    ps_int_coeff(a, n).abs = (a(n) / Real.from_rat(Rat.from_nat(n.suc))).abs
    abs_div(a(n), Real.from_rat(Rat.from_nat(n.suc)))
    suc_pos(n)
    Real.from_rat(Rat.from_nat(n.suc)) > Real.0
    Real.from_rat(Rat.from_nat(n.suc)) != Real.0
    (a(n) / Real.from_rat(Rat.from_nat(n.suc))).abs =
        a(n).abs / Real.from_rat(Rat.from_nat(n.suc)).abs
    not Real.from_rat(Rat.from_nat(n.suc)).is_negative
    Real.from_rat(Rat.from_nat(n.suc)).abs = Real.from_rat(Rat.from_nat(n.suc))
    (a(n) / Real.from_rat(Rat.from_nat(n.suc))).abs = a(n).abs / Real.from_rat(Rat.from_nat(n.suc))
    ps_int_coeff(a, n).abs = a(n).abs / Real.from_rat(Rat.from_nat(n.suc))
    Nat.1 <= n.suc
    from_nat_lte_mono(Nat.1, n.suc)
    from_nat[Real](Nat.1) <= from_nat[Real](n.suc)
    from_nat_one[Real]
    from_nat[Real](Nat.1) = Real.1
    from_nat_is_from_rat(n.suc)
    from_nat[Real](n.suc) = Real.from_rat(Rat.from_nat(n.suc))
    Real.1 <= Real.from_rat(Rat.from_nat(n.suc))
    abs_gte_zero(a(n))
    Real.0 <= a(n).abs
    div_nonneg_le_self(a(n).abs, Real.from_rat(Rat.from_nat(n.suc)))
    a(n).abs / Real.from_rat(Rat.from_nat(n.suc)) <= a(n).abs
    ps_int_coeff(a, n).abs <= a(n).abs
}

/// Each integrated term is dominated by |x| times the original term:
/// |a(n) x^(n+1) / (n + 1)| <= |x| * |a(n) x^n|.
theorem ps_int_term_abs_le(a: Nat -> Real, x: Real, n: Nat) {
    ps_int_term(a, x, n).abs <= x.abs * ps_term(a, x, n).abs
} by {
    ps_int_term(a, x, n) = a(n) * x.pow(n.suc) / Real.from_rat(Rat.from_nat(n.suc))
    ps_int_term(a, x, n).abs = (a(n) * x.pow(n.suc) / Real.from_rat(Rat.from_nat(n.suc))).abs
    abs_div(a(n) * x.pow(n.suc), Real.from_rat(Rat.from_nat(n.suc)))
    suc_pos(n)
    Real.from_rat(Rat.from_nat(n.suc)) > Real.0
    Real.from_rat(Rat.from_nat(n.suc)) != Real.0
    (a(n) * x.pow(n.suc) / Real.from_rat(Rat.from_nat(n.suc))).abs =
        (a(n) * x.pow(n.suc)).abs / Real.from_rat(Rat.from_nat(n.suc)).abs
    not Real.from_rat(Rat.from_nat(n.suc)).is_negative
    Real.from_rat(Rat.from_nat(n.suc)).abs = Real.from_rat(Rat.from_nat(n.suc))
    (a(n) * x.pow(n.suc) / Real.from_rat(Rat.from_nat(n.suc))).abs =
        (a(n) * x.pow(n.suc)).abs / Real.from_rat(Rat.from_nat(n.suc))
    mul_abs(a(n), x.pow(n.suc))
    (a(n) * x.pow(n.suc)).abs = a(n).abs * x.pow(n.suc).abs
    abs_pow(x, n.suc)
    x.pow(n.suc).abs = x.abs.pow(n.suc)
    (a(n) * x.pow(n.suc)).abs = a(n).abs * x.abs.pow(n.suc)
    pow_suc(x.abs, n)
    x.abs.pow(n.suc) = x.abs * x.abs.pow(n)
    (a(n) * x.pow(n.suc)).abs = a(n).abs * (x.abs * x.abs.pow(n))
    a(n).abs * (x.abs * x.abs.pow(n)) = a(n).abs * x.abs * x.abs.pow(n)
    (a(n) * x.pow(n.suc)).abs = a(n).abs * x.abs * x.abs.pow(n)
    (a(n) * x.pow(n.suc) / Real.from_rat(Rat.from_nat(n.suc))).abs =
        (a(n).abs * x.abs * x.abs.pow(n)) / Real.from_rat(Rat.from_nat(n.suc))
    ps_int_term(a, x, n).abs =
        (a(n).abs * x.abs * x.abs.pow(n)) / Real.from_rat(Rat.from_nat(n.suc))
    abs_gte_zero(a(n))
    Real.0 <= a(n).abs
    abs_gte_zero(x)
    Real.0 <= x.abs
    pow_nonneg(x.abs, n)
    Real.0 <= x.abs.pow(n)
    mul_nonneg(a(n).abs, x.abs)
    a(n).abs * x.abs >= Real.0
    Real.0 <= a(n).abs * x.abs
    mul_nonneg(a(n).abs * x.abs, x.abs.pow(n))
    (a(n).abs * x.abs) * x.abs.pow(n) >= Real.0
    Real.0 <= (a(n).abs * x.abs) * x.abs.pow(n)
    Real.0 <= a(n).abs * x.abs * x.abs.pow(n)
    Nat.1 <= n.suc
    from_nat_lte_mono(Nat.1, n.suc)
    from_nat[Real](Nat.1) <= from_nat[Real](n.suc)
    from_nat_one[Real]
    from_nat[Real](Nat.1) = Real.1
    from_nat_is_from_rat(n.suc)
    from_nat[Real](n.suc) = Real.from_rat(Rat.from_nat(n.suc))
    Real.1 <= Real.from_rat(Rat.from_nat(n.suc))
    div_nonneg_le_self(a(n).abs * x.abs * x.abs.pow(n), Real.from_rat(Rat.from_nat(n.suc)))
    (a(n).abs * x.abs * x.abs.pow(n)) / Real.from_rat(Rat.from_nat(n.suc)) <= a(n).abs * x.abs * x.abs.pow(n)
    ps_int_term(a, x, n).abs <= a(n).abs * x.abs * x.abs.pow(n)
    a(n).abs * x.abs * x.abs.pow(n) = x.abs * (a(n).abs * x.abs.pow(n))
    ps_int_term(a, x, n).abs <= x.abs * (a(n).abs * x.abs.pow(n))
    ps_term(a, x, n) = a(n) * x.pow(n)
    ps_term(a, x, n).abs = (a(n) * x.pow(n)).abs
    mul_abs(a(n), x.pow(n))
    (a(n) * x.pow(n)).abs = a(n).abs * x.pow(n).abs
    abs_pow(x, n)
    x.pow(n).abs = x.abs.pow(n)
    ps_term(a, x, n).abs = a(n).abs * x.abs.pow(n)
    x.abs * (a(n).abs * x.abs.pow(n)) = x.abs * ps_term(a, x, n).abs
    ps_int_term(a, x, n).abs <= x.abs * ps_term(a, x, n).abs
}

/// Term-by-term integration preserves absolute convergence: if the power series
/// with coefficients a converges absolutely at x, then so does the integrated
/// series (the integrated coefficients have the same radius of convergence).
theorem ps_int_term_abs_converges_of_abs_converges(a: Nat -> Real, x: Real) {
    absolutely_converges(ps_term(a, x)) implies absolutely_converges(ps_int_term(a, x))
} by {
    if absolutely_converges(ps_term(a, x)) {
        absolutely_converges(ps_term(a, x)) = converges(partial(abs_fn(ps_term(a, x))))
        converges(partial(abs_fn(ps_term(a, x))))
        forall(n: Nat) {
            ps_int_term_abs_le(a, x, n)
            ps_int_term(a, x, n).abs <= x.abs * ps_term(a, x, n).abs
            abs_fn(ps_int_term(a, x), n) = ps_int_term(a, x, n).abs
            abs_fn(ps_int_term(a, x), n) <= x.abs * ps_term(a, x, n).abs
            abs_fn(ps_term(a, x), n) = ps_term(a, x, n).abs
            mul_fn(x.abs, abs_fn(ps_term(a, x)))(n) = x.abs * abs_fn(ps_term(a, x), n)
            mul_fn(x.abs, abs_fn(ps_term(a, x)))(n) = x.abs * ps_term(a, x, n).abs
            abs_fn(ps_int_term(a, x), n) <= mul_fn(x.abs, abs_fn(ps_term(a, x)))(n)
        }
        seq_lte(abs_fn(ps_int_term(a, x)), mul_fn(x.abs, abs_fn(ps_term(a, x)))) =
            forall(n: Nat) {
                abs_fn(ps_int_term(a, x), n) <= mul_fn(x.abs, abs_fn(ps_term(a, x)))(n)
            }
        seq_lte(abs_fn(ps_int_term(a, x)), mul_fn(x.abs, abs_fn(ps_term(a, x))))
        forall(n: Nat) {
            abs_gte_zero(x)
            Real.0 <= x.abs
            abs_gte_zero(ps_term(a, x, n))
            Real.0 <= ps_term(a, x, n).abs
            mul_nonneg(x.abs, ps_term(a, x, n).abs)
            x.abs * ps_term(a, x, n).abs >= Real.0
            Real.0 <= x.abs * ps_term(a, x, n).abs
            abs_fn(ps_term(a, x), n) = ps_term(a, x, n).abs
            mul_fn(x.abs, abs_fn(ps_term(a, x)))(n) = x.abs * abs_fn(ps_term(a, x), n)
            mul_fn(x.abs, abs_fn(ps_term(a, x)))(n) = x.abs * ps_term(a, x, n).abs
            Real.0 <= mul_fn(x.abs, abs_fn(ps_term(a, x)))(n)
        }
        is_lower_bound(mul_fn(x.abs, abs_fn(ps_term(a, x))), Real.0) =
            forall(n: Nat) {
                Real.0 <= mul_fn(x.abs, abs_fn(ps_term(a, x)))(n)
            }
        is_lower_bound(mul_fn(x.abs, abs_fn(ps_term(a, x))), Real.0)
        converges_mul_seq(x.abs, partial(abs_fn(ps_term(a, x))))
        converges(mul_seq(x.abs, partial(abs_fn(ps_term(a, x)))))
        forall(n: Nat) {
            partial_mul_seq_comm(x.abs, abs_fn(ps_term(a, x)))
            partial(mul_seq(x.abs, abs_fn(ps_term(a, x))), n) =
                mul_seq(x.abs, partial(abs_fn(ps_term(a, x))), n)
            forall(k: Nat) {
                if k < n {
                    mul_seq(x.abs, abs_fn(ps_term(a, x)), k) = x.abs * abs_fn(ps_term(a, x), k)
                    mul_fn(x.abs, abs_fn(ps_term(a, x)))(k) = x.abs * abs_fn(ps_term(a, x), k)
                    mul_seq(x.abs, abs_fn(ps_term(a, x)), k) = mul_fn(x.abs, abs_fn(ps_term(a, x)))(k)
                }
            }
            partial_pointwise_eq(mul_seq(x.abs, abs_fn(ps_term(a, x))),
                mul_fn(x.abs, abs_fn(ps_term(a, x))), n)
            partial(mul_seq(x.abs, abs_fn(ps_term(a, x))), n) =
                partial(mul_fn(x.abs, abs_fn(ps_term(a, x))), n)
            partial(mul_fn(x.abs, abs_fn(ps_term(a, x))), n) =
                mul_seq(x.abs, partial(abs_fn(ps_term(a, x))), n)
        }
        converges_pointwise_eq(mul_seq(x.abs, partial(abs_fn(ps_term(a, x)))),
            partial(mul_fn(x.abs, abs_fn(ps_term(a, x)))))
        converges(partial(mul_fn(x.abs, abs_fn(ps_term(a, x)))))
        absolutely_converges_comparison(ps_int_term(a, x), mul_fn(x.abs, abs_fn(ps_term(a, x))))
        absolutely_converges(ps_int_term(a, x))
    }
}

// ---------------------------------------------------------------------------
// The Mercator series for ln(1 + x).
// ---------------------------------------------------------------------------

/// The nth term of the Mercator series for ln(1 + x): (-1)^n x^(n+1) / (n + 1).
define mercator_term(x: Real, n: Nat) -> Real {
    alternating_sign[Real](n) * x.pow(n.suc) / Real.from_rat(Rat.from_nat(n.suc))
}

/// The sum of the Mercator series at x, when the series converges.
define mercator_sum(x: Real) -> Real {
    limit(partial(mercator_term(x)))
}

/// The nth term of the differentiated Mercator series: (-1)^n x^n.
/// This is the derivative of the nth Mercator term.
define mercator_diff_term(x: Real, n: Nat) -> Real {
    alternating_sign[Real](n) * x.pow(n)
}

/// The absolute value of the nth Mercator term is |x|^(n+1) / (n + 1).
theorem mercator_term_abs(x: Real, n: Nat) {
    mercator_term(x, n).abs = x.abs.pow(n.suc) / Real.from_rat(Rat.from_nat(n.suc))
} by {
    mercator_term(x, n) = alternating_sign[Real](n) * x.pow(n.suc) / Real.from_rat(Rat.from_nat(n.suc))
    mercator_term(x, n).abs =
        (alternating_sign[Real](n) * x.pow(n.suc) / Real.from_rat(Rat.from_nat(n.suc))).abs
    abs_div(alternating_sign[Real](n) * x.pow(n.suc), Real.from_rat(Rat.from_nat(n.suc)))
    suc_pos(n)
    Real.from_rat(Rat.from_nat(n.suc)) > Real.0
    Real.from_rat(Rat.from_nat(n.suc)) != Real.0
    (alternating_sign[Real](n) * x.pow(n.suc) / Real.from_rat(Rat.from_nat(n.suc))).abs =
        (alternating_sign[Real](n) * x.pow(n.suc)).abs / Real.from_rat(Rat.from_nat(n.suc)).abs
    not Real.from_rat(Rat.from_nat(n.suc)).is_negative
    Real.from_rat(Rat.from_nat(n.suc)).abs = Real.from_rat(Rat.from_nat(n.suc))
    (alternating_sign[Real](n) * x.pow(n.suc) / Real.from_rat(Rat.from_nat(n.suc))).abs =
        (alternating_sign[Real](n) * x.pow(n.suc)).abs / Real.from_rat(Rat.from_nat(n.suc))
    mul_abs(alternating_sign[Real](n), x.pow(n.suc))
    (alternating_sign[Real](n) * x.pow(n.suc)).abs =
        alternating_sign[Real](n).abs * x.pow(n.suc).abs
    alternating_sign_abs(n)
    alternating_sign[Real](n).abs = Real.1
    alternating_sign[Real](n).abs * x.pow(n.suc).abs = Real.1 * x.pow(n.suc).abs
    Real.1 * x.pow(n.suc).abs = x.pow(n.suc).abs
    (alternating_sign[Real](n) * x.pow(n.suc)).abs = x.pow(n.suc).abs
    abs_pow(x, n.suc)
    x.pow(n.suc).abs = x.abs.pow(n.suc)
    (alternating_sign[Real](n) * x.pow(n.suc)).abs = x.abs.pow(n.suc)
    mercator_term(x, n).abs = x.abs.pow(n.suc) / Real.from_rat(Rat.from_nat(n.suc))
}

/// Each Mercator term is dominated by the geometric term |x| * |x|^n.
theorem mercator_term_abs_le_geom(x: Real, n: Nat) {
    mercator_term(x, n).abs <= x.abs * x.abs.pow(n)
} by {
    mercator_term_abs(x, n)
    mercator_term(x, n).abs = x.abs.pow(n.suc) / Real.from_rat(Rat.from_nat(n.suc))
    pow_suc(x.abs, n)
    x.abs.pow(n.suc) = x.abs * x.abs.pow(n)
    x.abs.pow(n.suc) / Real.from_rat(Rat.from_nat(n.suc)) =
        (x.abs * x.abs.pow(n)) / Real.from_rat(Rat.from_nat(n.suc))
    abs_gte_zero(x)
    Real.0 <= x.abs
    pow_nonneg(x.abs, n)
    Real.0 <= x.abs.pow(n)
    mul_nonneg(x.abs, x.abs.pow(n))
    x.abs * x.abs.pow(n) >= Real.0
    Real.0 <= x.abs * x.abs.pow(n)
    Nat.1 <= n.suc
    from_nat_lte_mono(Nat.1, n.suc)
    from_nat[Real](Nat.1) <= from_nat[Real](n.suc)
    from_nat_one[Real]
    from_nat[Real](Nat.1) = Real.1
    from_nat_is_from_rat(n.suc)
    from_nat[Real](n.suc) = Real.from_rat(Rat.from_nat(n.suc))
    Real.1 <= Real.from_rat(Rat.from_nat(n.suc))
    div_nonneg_le_self(x.abs * x.abs.pow(n), Real.from_rat(Rat.from_nat(n.suc)))
    (x.abs * x.abs.pow(n)) / Real.from_rat(Rat.from_nat(n.suc)) <= x.abs * x.abs.pow(n)
    x.abs.pow(n.suc) / Real.from_rat(Rat.from_nat(n.suc)) <= x.abs * x.abs.pow(n)
    mercator_term(x, n).abs <= x.abs * x.abs.pow(n)
}

/// The Mercator series converges absolutely for |x| < 1.
theorem mercator_term_abs_converges_small(x: Real) {
    x.abs < Real.1 implies absolutely_converges(mercator_term(x))
} by {
    if x.abs < Real.1 {
        forall(n: Nat) {
            mercator_term_abs_le_geom(x, n)
            mercator_term(x, n).abs <= x.abs * x.abs.pow(n)
            abs_fn(mercator_term(x), n) = mercator_term(x, n).abs
            abs_fn(mercator_term(x), n) <= x.abs * x.abs.pow(n)
            mul_fn(x.abs, x.abs.pow)(n) = x.abs * x.abs.pow(n)
            abs_fn(mercator_term(x), n) <= mul_fn(x.abs, x.abs.pow)(n)
        }
        seq_lte(abs_fn(mercator_term(x)), mul_fn(x.abs, x.abs.pow)) =
            forall(n: Nat) {
                abs_fn(mercator_term(x), n) <= mul_fn(x.abs, x.abs.pow)(n)
            }
        seq_lte(abs_fn(mercator_term(x)), mul_fn(x.abs, x.abs.pow))
        forall(n: Nat) {
            abs_gte_zero(x)
            Real.0 <= x.abs
            pow_nonneg(x.abs, n)
            Real.0 <= x.abs.pow(n)
            mul_nonneg(x.abs, x.abs.pow(n))
            x.abs * x.abs.pow(n) >= Real.0
            Real.0 <= x.abs * x.abs.pow(n)
            mul_fn(x.abs, x.abs.pow)(n) = x.abs * x.abs.pow(n)
            Real.0 <= mul_fn(x.abs, x.abs.pow)(n)
        }
        is_lower_bound(mul_fn(x.abs, x.abs.pow), Real.0) =
            forall(n: Nat) {
                Real.0 <= mul_fn(x.abs, x.abs.pow)(n)
            }
        is_lower_bound(mul_fn(x.abs, x.abs.pow), Real.0)
        x.abs >= Real.0
        x.abs.abs = x.abs
        x.abs.abs < Real.1
        geom_converges(x.abs)
        converges(partial(x.abs.pow))
        converges_mul_seq(x.abs, partial(x.abs.pow))
        converges(mul_seq(x.abs, partial(x.abs.pow)))
        forall(n: Nat) {
            partial_mul_seq_comm(x.abs, x.abs.pow)
            partial(mul_seq(x.abs, x.abs.pow), n) = mul_seq(x.abs, partial(x.abs.pow), n)
            forall(k: Nat) {
                if k < n {
                    mul_seq(x.abs, x.abs.pow, k) = x.abs * x.abs.pow(k)
                    mul_fn(x.abs, x.abs.pow)(k) = x.abs * x.abs.pow(k)
                    mul_seq(x.abs, x.abs.pow, k) = mul_fn(x.abs, x.abs.pow)(k)
                }
            }
            partial_pointwise_eq(mul_seq(x.abs, x.abs.pow), mul_fn(x.abs, x.abs.pow), n)
            partial(mul_seq(x.abs, x.abs.pow), n) = partial(mul_fn(x.abs, x.abs.pow), n)
            partial(mul_fn(x.abs, x.abs.pow), n) = mul_seq(x.abs, partial(x.abs.pow), n)
        }
        converges_pointwise_eq(mul_seq(x.abs, partial(x.abs.pow)), partial(mul_fn(x.abs, x.abs.pow)))
        converges(partial(mul_fn(x.abs, x.abs.pow)))
        absolutely_converges_comparison(mercator_term(x), mul_fn(x.abs, x.abs.pow))
        absolutely_converges(mercator_term(x))
    }
}

/// Differentiating the Mercator coefficients cancels the denominators:
/// (n + 1) * (1 / (n + 1)) = 1.
theorem mercator_diff_coeff(n: Nat) {
    Real.from_rat(Rat.from_nat(n.suc)) * (Real.1 / Real.from_rat(Rat.from_nat(n.suc))) = Real.1
} by {
    suc_pos(n)
    Real.from_rat(Rat.from_nat(n.suc)) > Real.0
    Real.from_rat(Rat.from_nat(n.suc)) != Real.0
    mul_div_cancel(Real.1, Real.from_rat(Rat.from_nat(n.suc)))
    Real.from_rat(Rat.from_nat(n.suc)) * (Real.1 / Real.from_rat(Rat.from_nat(n.suc))) = Real.1
}

/// The nth differentiated Mercator term is the nth power of -x: (-1)^n x^n = (-x)^n.
theorem mercator_diff_term_eq_alt_pow(x: Real, n: Nat) {
    mercator_diff_term(x, n) = (-x).pow(n)
} by {
    mercator_diff_term(x, n) = alternating_sign[Real](n) * x.pow(n)
    real_pow_mul_distrib(-Real.1, x, n)
    (-Real.1 * x).pow(n) = (-Real.1).pow(n) * x.pow(n)
    mul_neg_one_left(x)
    -Real.1 * x = -x
    (-x).pow(n) = (-Real.1).pow(n) * x.pow(n)
    alternating_sign_eq_neg_one_pow[Real](n)
    alternating_sign[Real](n) = (-Real.1).pow(n)
    (-Real.1).pow(n) = alternating_sign[Real](n)
    (-Real.1).pow(n) * x.pow(n) = alternating_sign[Real](n) * x.pow(n)
    (-x).pow(n) = alternating_sign[Real](n) * x.pow(n)
    alternating_sign[Real](n) * x.pow(n) = (-x).pow(n)
    mercator_diff_term(x, n) = (-x).pow(n)
}

/// The alternating geometric series sums to 1 / (1 + x) for |x| < 1:
/// sum_{n>=0} (-x)^n = 1 / (1 + x).
theorem alt_geom_series_eq_recip(x: Real) {
    x.abs < Real.1 implies limit(partial((-x).pow)) = Real.1 / (Real.1 + x)
} by {
    if x.abs < Real.1 {
        abs_neg(x)
        (-x).abs = x.abs
        (-x).abs < Real.1
        geom_converges(-x)
        converges(partial((-x).pow))
        geom_series_no_div(-x)
        Real.1 + (-x) * limit(partial((-x).pow)) = limit(partial((-x).pow))
        let g = limit(partial((-x).pow))
        Real.1 + (-x) * g = g
        Real.1 + (-x) * g - (-x) * g = g - (-x) * g
        Real.1 = g - (-x) * g
        (-x) * g = -(x * g)
        g - (-x) * g = g + x * g
        Real.1 = g + x * g
        g + x * g = (Real.1 + x) * g
        Real.1 = (Real.1 + x) * g
        lte_abs(-x)
        -x <= (-x).abs
        -x <= x.abs
        lte_lt_trans(-x, x.abs, Real.1)
        -x < Real.1
        lt_add_right(-x, Real.1, x)
        -x + x < Real.1 + x
        -x + x = Real.0
        Real.0 < Real.1 + x
        Real.1 + x > Real.0
        Real.1 + x != Real.0
        ((Real.1 + x) * g) / (Real.1 + x) = Real.1 / (Real.1 + x)
        div_mul_cancel_left(Real.1 + x, g)
        ((Real.1 + x) * g) / (Real.1 + x) = g
        g = Real.1 / (Real.1 + x)
        limit(partial((-x).pow)) = Real.1 / (Real.1 + x)
    }
}

/// The Mercator series differentiates term by term to the alternating geometric
/// series: sum_{n>=0} (-1)^n x^n = 1 / (1 + x) for |x| < 1.
theorem mercator_diff_series_eq_recip(x: Real) {
    x.abs < Real.1 implies limit(partial(mercator_diff_term(x))) = Real.1 / (Real.1 + x)
} by {
    if x.abs < Real.1 {
        forall(n: Nat) {
            forall(k: Nat) {
                if k < n {
                    mercator_diff_term_eq_alt_pow(x, k)
                    mercator_diff_term(x, k) = (-x).pow(k)
                }
            }
            partial_pointwise_eq(mercator_diff_term(x), (-x).pow, n)
            partial(mercator_diff_term(x), n) = partial((-x).pow, n)
            partial((-x).pow, n) = partial(mercator_diff_term(x), n)
        }
        abs_neg(x)
        (-x).abs = x.abs
        (-x).abs < Real.1
        geom_converges(-x)
        converges(partial((-x).pow))
        limit_pointwise_eq(partial((-x).pow), partial(mercator_diff_term(x)))
        limit(partial(mercator_diff_term(x))) = limit(partial((-x).pow))
        alt_geom_series_eq_recip(x)
        limit(partial((-x).pow)) = Real.1 / (Real.1 + x)
        limit(partial(mercator_diff_term(x))) = Real.1 / (Real.1 + x)
    }
}

/// The Mercator series at x = 1 is the alternating harmonic series, which
/// converges: sum_{n>=0} (-1)^n / (n + 1) is convergent.
theorem mercator_series_at_one_converges {
    converges(partial(mercator_term(Real.1)))
} by {
    forall(n: Nat) {
        mercator_term(Real.1, n) =
            alternating_sign[Real](n) * Real.1.pow(n.suc) / Real.from_rat(Rat.from_nat(n.suc))
        one_pow[Real](n.suc)
        Real.1.pow(n.suc) = Real.1
        alternating_sign[Real](n) * Real.1 / Real.from_rat(Rat.from_nat(n.suc)) =
            alternating_sign[Real](n) * (Real.1 / Real.from_rat(Rat.from_nat(n.suc)))
        mercator_term(Real.1, n) =
            alternating_sign[Real](n) * (Real.1 / Real.from_rat(Rat.from_nat(n.suc)))
        alt_term(harmonic)(n) = alternating_sign[Real](n) * harmonic(n)
        harmonic(n) = Real.1 / from_nat[Real](n.suc)
        from_nat_is_from_rat(n.suc)
        from_nat[Real](n.suc) = Real.from_rat(Rat.from_nat(n.suc))
        harmonic(n) = Real.1 / Real.from_rat(Rat.from_nat(n.suc))
        alt_term(harmonic)(n) = alternating_sign[Real](n) * (Real.1 / Real.from_rat(Rat.from_nat(n.suc)))
        mercator_term(Real.1, n) = alt_term(harmonic)(n)
        alt_term(harmonic)(n) = mercator_term(Real.1, n)
    }
    harmonic_decreasing
    is_decreasing_seq(harmonic)
    one_over_suc_converges
    converges(harmonic)
    limit_one_over_suc
    limit(harmonic) = Real.0
    vanishes(harmonic) = converges(harmonic) and limit(harmonic) = Real.0
    vanishes(harmonic)
    alternating_series_test(harmonic)
    converges(partial(alt_term(harmonic)))
    converges_pointwise_eq(alt_term(harmonic), mercator_term(Real.1))
    converges(partial(mercator_term(Real.1)))
}
