from data.basic.set import Set, compl_of_compl_is_self
from real.real_field import Real
from real.topology import closure, interior, is_closed_set, is_open_set
from real.topology_closure import closure_is_closed
from real.topology_closure_interior_duality import closure_complement_eq_interior_complement,
    interior_complement_eq_closure_complement, interior_eq_closure_complement_complement
from real.topology_clopen import clopen_real_set_eq_closure, clopen_real_set_eq_interior,
    clopen_real_set_is_closed, clopen_real_set_is_open, is_clopen_real_set
from real.topology_interior_algebra import interior_mono, open_set_eq_interior
from real.topology_interior_closure_idempotent import closed_set_iff_eq_closure, closure_empty_real_set,
    closure_universal_real_set, interior_empty_real_set, interior_is_open,
    interior_universal_real_set

/// The regular-open hull of a real set.
define regular_open_hull(s: Set[Real]) -> Set[Real] {
    interior(closure(s))
}

/// The regular-closed hull of a real set.
define regular_closed_hull(s: Set[Real]) -> Set[Real] {
    closure(interior(s))
}

/// True if a real set is equal to the interior of its closure.
define is_regular_open_real_set(s: Set[Real]) -> Bool {
    s = regular_open_hull(s)
}

/// True if a real set is equal to the closure of its interior.
define is_regular_closed_real_set(s: Set[Real]) -> Bool {
    s = regular_closed_hull(s)
}

/// Membership in the regular-open hull is membership in the interior of the closure.
theorem regular_open_hull_contains_eq(s: Set[Real], x: Real) {
    regular_open_hull(s).contains(x) = interior(closure(s)).contains(x)
}

/// Membership in the regular-closed hull is membership in the closure of the interior.
theorem regular_closed_hull_contains_eq(s: Set[Real], x: Real) {
    regular_closed_hull(s).contains(x) = closure(interior(s)).contains(x)
}

/// A regular-open hull is open.
theorem regular_open_hull_is_open(s: Set[Real]) {
    is_open_set(regular_open_hull(s))
} by {
    interior_is_open(closure(s))
    is_open_set(interior(closure(s)))
    is_open_set(regular_open_hull(s))
}

/// A regular-closed hull is closed.
theorem regular_closed_hull_is_closed(s: Set[Real]) {
    is_closed_set(regular_closed_hull(s))
} by {
    closure_is_closed(interior(s))
    is_closed_set(closure(interior(s)))
    is_closed_set(regular_closed_hull(s))
}

/// A regular-open real set is open.
theorem regular_open_real_set_is_open(s: Set[Real]) {
    is_regular_open_real_set(s) implies is_open_set(s)
} by {
    if is_regular_open_real_set(s) {
        is_regular_open_real_set(s) = (s = regular_open_hull(s))
        s = regular_open_hull(s)
        regular_open_hull_is_open(s)
        is_open_set(regular_open_hull(s))
        is_open_set(s)
    }
}

/// A regular-closed real set is closed.
theorem regular_closed_real_set_is_closed(s: Set[Real]) {
    is_regular_closed_real_set(s) implies is_closed_set(s)
} by {
    if is_regular_closed_real_set(s) {
        is_regular_closed_real_set(s) = (s = regular_closed_hull(s))
        s = regular_closed_hull(s)
        regular_closed_hull_is_closed(s)
        is_closed_set(regular_closed_hull(s))
        is_closed_set(s)
    }
}

/// A set is regular open exactly when it equals the regular-open hull.
theorem regular_open_real_set_eq_hull(s: Set[Real]) {
    is_regular_open_real_set(s) = (s = regular_open_hull(s))
}

/// A set is regular closed exactly when it equals the regular-closed hull.
theorem regular_closed_real_set_eq_hull(s: Set[Real]) {
    is_regular_closed_real_set(s) = (s = regular_closed_hull(s))
}

/// The empty real set is regular open.
theorem empty_real_set_is_regular_open {
    is_regular_open_real_set(Set[Real].empty_set)
} by {
    closure_empty_real_set
    closure(Set[Real].empty_set) = Set[Real].empty_set
    interior_empty_real_set
    interior(Set[Real].empty_set) = Set[Real].empty_set
    regular_open_hull(Set[Real].empty_set) = Set[Real].empty_set
    Set[Real].empty_set = regular_open_hull(Set[Real].empty_set)
    is_regular_open_real_set(Set[Real].empty_set)
}

/// The empty real set is regular closed.
theorem empty_real_set_is_regular_closed {
    is_regular_closed_real_set(Set[Real].empty_set)
} by {
    interior_empty_real_set
    interior(Set[Real].empty_set) = Set[Real].empty_set
    closure_empty_real_set
    closure(Set[Real].empty_set) = Set[Real].empty_set
    regular_closed_hull(Set[Real].empty_set) = Set[Real].empty_set
    Set[Real].empty_set = regular_closed_hull(Set[Real].empty_set)
    is_regular_closed_real_set(Set[Real].empty_set)
}

/// The universal real set is regular open.
theorem universal_real_set_is_regular_open {
    is_regular_open_real_set(Set[Real].universal_set)
} by {
    closure_universal_real_set
    closure(Set[Real].universal_set) = Set[Real].universal_set
    interior_universal_real_set
    interior(Set[Real].universal_set) = Set[Real].universal_set
    regular_open_hull(Set[Real].universal_set) = Set[Real].universal_set
    Set[Real].universal_set = regular_open_hull(Set[Real].universal_set)
    is_regular_open_real_set(Set[Real].universal_set)
}

/// The universal real set is regular closed.
theorem universal_real_set_is_regular_closed {
    is_regular_closed_real_set(Set[Real].universal_set)
} by {
    interior_universal_real_set
    interior(Set[Real].universal_set) = Set[Real].universal_set
    closure_universal_real_set
    closure(Set[Real].universal_set) = Set[Real].universal_set
    regular_closed_hull(Set[Real].universal_set) = Set[Real].universal_set
    Set[Real].universal_set = regular_closed_hull(Set[Real].universal_set)
    is_regular_closed_real_set(Set[Real].universal_set)
}

/// An open real set is contained in its regular-open hull.
theorem open_set_subset_regular_open_hull(s: Set[Real]) {
    is_open_set(s) implies s.subset(regular_open_hull(s))
} by {
    if is_open_set(s) {
        open_set_eq_interior(s)
        s = interior(s)
        from real.topology import set_subset_closure
        set_subset_closure(s)
        s.subset(closure(s))
        interior_mono(s, closure(s))
        interior(s).subset(interior(closure(s)))
        s.subset(interior(closure(s)))
        s.subset(regular_open_hull(s))
    }
}

/// The regular-open hull is contained in the closure of the set.
theorem regular_open_hull_subset_closure(s: Set[Real]) {
    regular_open_hull(s).subset(closure(s))
} by {
    from real.topology import interior_subset
    interior_subset(closure(s))
    interior(closure(s)).subset(closure(s))
    regular_open_hull(s).subset(closure(s))
}

/// The interior of a set is contained in its regular-closed hull.
theorem interior_subset_regular_closed_hull(s: Set[Real]) {
    interior(s).subset(regular_closed_hull(s))
} by {
    from real.topology import set_subset_closure
    set_subset_closure(interior(s))
    interior(s).subset(closure(interior(s)))
    interior(s).subset(regular_closed_hull(s))
}

/// The regular-closed hull is contained in the closure of the set.
theorem regular_closed_hull_subset_closure(s: Set[Real]) {
    regular_closed_hull(s).subset(closure(s))
} by {
    from real.topology import closure_mono, interior_subset
    interior_subset(s)
    interior(s).subset(s)
    closure_mono(interior(s), s)
    closure(interior(s)).subset(closure(s))
    regular_closed_hull(s).subset(closure(s))
}

/// The complement of a regular-open real set is regular closed.
theorem complement_of_regular_open_real_set_is_regular_closed(s: Set[Real]) {
    is_regular_open_real_set(s) implies is_regular_closed_real_set(s.c)
} by {
    if is_regular_open_real_set(s) {
        s = regular_open_hull(s)
        s = interior(closure(s))
        s.c = interior(closure(s)).c
        closure_complement_eq_interior_complement(closure(s))
        closure(closure(s).c) = interior(closure(s)).c
        interior_complement_eq_closure_complement(s)
        interior(s.c) = closure(s).c
        closure(interior(s.c)) = closure(closure(s).c)
        regular_closed_hull(s.c) = closure(interior(s.c))
        s.c = regular_closed_hull(s.c)
        is_regular_closed_real_set(s.c)
    }
}

/// The complement of a regular-closed real set is regular open.
theorem complement_of_regular_closed_real_set_is_regular_open(s: Set[Real]) {
    is_regular_closed_real_set(s) implies is_regular_open_real_set(s.c)
} by {
    if is_regular_closed_real_set(s) {
        s = regular_closed_hull(s)
        s = closure(interior(s))
        s.c = closure(interior(s)).c
        interior_eq_closure_complement_complement(interior(s))
        interior(interior(s)) = closure(interior(s).c).c
        closure(interior(s)).c = interior(interior(s).c)
        closure_complement_eq_interior_complement(s.c)
        closure(s.c.c) = interior(s.c).c
        compl_of_compl_is_self[Real](s)
        s.c.c = s
        closure(s) = interior(s.c).c
        closure(s).c = interior(s.c).c.c
        compl_of_compl_is_self[Real](interior(s.c))
        interior(s.c).c.c = interior(s.c)
        closure(s).c = interior(s.c)
        interior(s).c = closure(s.c)
        interior(interior(s).c) = interior(closure(s.c))
        regular_open_hull(s.c) = interior(closure(s.c))
        s.c = regular_open_hull(s.c)
        is_regular_open_real_set(s.c)
    }
}

/// Regular openness is dual to regular closedness of the complement.
theorem regular_open_complement_eq_regular_closed(s: Set[Real]) {
    is_regular_open_real_set(s.c) = is_regular_closed_real_set(s)
} by {
    if is_regular_open_real_set(s.c) {
        complement_of_regular_open_real_set_is_regular_closed(s.c)
        is_regular_closed_real_set(s.c.c)
        compl_of_compl_is_self[Real](s)
        s.c.c = s
        is_regular_closed_real_set(s)
    }
    if is_regular_closed_real_set(s) {
        complement_of_regular_closed_real_set_is_regular_open(s)
        is_regular_open_real_set(s.c)
    }
    is_regular_open_real_set(s.c) = is_regular_closed_real_set(s)
}

/// Regular closedness is dual to regular openness of the complement.
theorem regular_closed_complement_eq_regular_open(s: Set[Real]) {
    is_regular_closed_real_set(s.c) = is_regular_open_real_set(s)
} by {
    if is_regular_closed_real_set(s.c) {
        complement_of_regular_closed_real_set_is_regular_open(s.c)
        is_regular_open_real_set(s.c.c)
        compl_of_compl_is_self[Real](s)
        s.c.c = s
        is_regular_open_real_set(s)
    }
    if is_regular_open_real_set(s) {
        complement_of_regular_open_real_set_is_regular_closed(s)
        is_regular_closed_real_set(s.c)
    }
    is_regular_closed_real_set(s.c) = is_regular_open_real_set(s)
}

/// A clopen real set is regular open.
theorem clopen_real_set_is_regular_open(s: Set[Real]) {
    is_clopen_real_set(s) implies is_regular_open_real_set(s)
} by {
    if is_clopen_real_set(s) {
        clopen_real_set_is_open(s)
        clopen_real_set_eq_closure(s)
        is_open_set(s)
        s = closure(s)
        open_set_eq_interior(s)
        s = interior(s)
        interior(closure(s)) = interior(s)
        regular_open_hull(s) = s
        s = regular_open_hull(s)
        is_regular_open_real_set(s)
    }
}

/// A clopen real set is regular closed.
theorem clopen_real_set_is_regular_closed(s: Set[Real]) {
    is_clopen_real_set(s) implies is_regular_closed_real_set(s)
} by {
    if is_clopen_real_set(s) {
        clopen_real_set_is_closed(s)
        clopen_real_set_eq_interior(s)
        is_closed_set(s)
        s = interior(s)
        closed_set_iff_eq_closure(s)
        s = closure(s)
        closure(interior(s)) = closure(s)
        regular_closed_hull(s) = s
        s = regular_closed_hull(s)
        is_regular_closed_real_set(s)
    }
}
