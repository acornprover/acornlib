/// Bounded and vanishing real-sequence algebra for asymptotic estimates.
from nat import Nat
from real.real_base import Real, lte_lt_trans, abs_gte_zero, abs_not_neg, lt_add_pos, add_lt_lt, add_zero_right, pos_gt_zero, gt_zero_imp_pos
from real.real_seq import converges, converges_to, limit, add_seq, tail_bound, tail_bound_implies_is_close, converges_to_unique, converges_imp_converges_to, limit_add_seq
from real.real_series import seq_lte, mul_seq, neg_seq, const_seq, converges_const_seq,
    neg_seq_converges, neg_seq_converges_to, limit_neg_seq, triangle_ineq, add_seq_converges
from real.prod_seq import prod_seq, prod_seq_converges, limit_of_prod, mul_le_lt_of_nonneg_pos
from real.real_ring import lte_mul_nonneg_left, mul_abs, gt_pos_is_pos, real_mul_comm, zero_lte_imp_non_neg, non_neg_imp_zero_lte, mul_nonneg, exists_small_mul_variant_2
from order import lt_imp_lte
from real.limits import vanishes, abs_seq, vanishes_abs_seq, vanishes_of_abs_lte_abs, vanishes_prod_seq
from real.bounded_seq import is_bounded_seq, converges_imp_bounded_seq, converges_to_imp_bounded_seq

numerals Real

/// A bounded real sequence admits a positive absolute-value bound.
theorem bounded_seq_positive_bound(a: Nat -> Real) {
    is_bounded_seq(a) implies exists(bound: Real) {
        bound.is_positive and forall(n: Nat) {
            a(n).abs < bound
        }
    }
} by {
    if is_bounded_seq(a) {
        let bound: Real satisfy {
            forall(n: Nat) {
                a(n).abs < bound
            }
        }
        abs_gte_zero(a(Nat.0))
        a(Nat.0).abs >= Real.0
        Real.0 <= a(Nat.0).abs
        a(Nat.0).abs < bound
        lte_lt_trans(Real.0, a(Nat.0).abs, bound)
        Real.0 < bound
        bound.is_positive
        exists(bound2: Real) {
            bound2.is_positive and forall(n: Nat) {
                a(n).abs < bound2
            }
        }
    }
}

/// A constant real sequence is bounded.
theorem const_seq_bounded(c: Real) {
    is_bounded_seq(const_seq(c))
} by {
    let bound = c.abs + Real.1
    Real.1.is_positive
    c.abs < bound
    forall(n: Nat) {
        const_seq(c, n) = c
        const_seq(c)(n).abs = c.abs
        const_seq(c)(n).abs < bound
    }
}

/// A convergent real sequence is bounded, as a reusable asymptotic endpoint.
theorem convergent_seq_bounded(a: Nat -> Real) {
    converges(a) implies is_bounded_seq(a)
} by {
    converges_imp_bounded_seq(a)
}

/// A real sequence converging to a specified value is bounded.
theorem converges_to_seq_bounded(a: Nat -> Real, x: Real) {
    converges_to(a, x) implies is_bounded_seq(a)
} by {
    converges_to_imp_bounded_seq(a, x)
}

/// Termwise negation preserves boundedness.
theorem bounded_neg_seq(a: Nat -> Real) {
    is_bounded_seq(a) implies is_bounded_seq(neg_seq(a))
} by {
    if is_bounded_seq(a) {
        let bound: Real satisfy {
            forall(n: Nat) {
                a(n).abs < bound
            }
        }
        forall(n: Nat) {
            neg_seq(a, n) = -a(n)
            (-a(n)).abs = a(n).abs
            neg_seq(a)(n).abs = a(n).abs
            neg_seq(a)(n).abs < bound
        }
    }
}

/// Termwise absolute value preserves boundedness.
theorem bounded_abs_seq(a: Nat -> Real) {
    is_bounded_seq(a) implies is_bounded_seq(abs_seq(a))
} by {
    if is_bounded_seq(a) {
        let bound: Real satisfy {
            forall(n: Nat) {
                a(n).abs < bound
            }
        }
        forall(n: Nat) {
            abs_seq(a, n) = a(n).abs
            abs_seq(a)(n).abs = a(n).abs
            abs_seq(a)(n).abs < bound
        }
    }
}

/// A sequence dominated in absolute value by a bounded sequence is bounded.
theorem bounded_of_abs_lte_abs(a: Nat -> Real, b: Nat -> Real) {
    seq_lte(abs_seq(a), abs_seq(b)) and is_bounded_seq(b) implies is_bounded_seq(a)
} by {
    if seq_lte(abs_seq(a), abs_seq(b)) and is_bounded_seq(b) {
        let bound: Real satisfy {
            forall(n: Nat) {
                b(n).abs < bound
            }
        }
        forall(n: Nat) {
            abs_seq(a, n) <= abs_seq(b, n)
            abs_seq(a, n) = a(n).abs
            abs_seq(b, n) = b(n).abs
            a(n).abs <= b(n).abs
            b(n).abs < bound
            lte_lt_trans(a(n).abs, b(n).abs, bound)
            a(n).abs < bound
        }
    }
}

/// Scalar multiplication preserves boundedness.
theorem bounded_mul_seq(c: Real, a: Nat -> Real) {
    is_bounded_seq(a) implies is_bounded_seq(mul_seq(c, a))
} by {
    if is_bounded_seq(a) {
        bounded_seq_positive_bound(a)
        let base_bound: Real satisfy {
            base_bound.is_positive and forall(n: Nat) {
                a(n).abs < base_bound
            }
        }
        let bound = c.abs * base_bound + Real.1
        not c.abs.is_negative
        non_neg_imp_zero_lte(c.abs)
        Real.0 <= c.abs
        base_bound.is_positive
        pos_gt_zero(base_bound)
        base_bound > Real.0
        lt_imp_lte[Real](Real.0, base_bound)
        Real.0 <= base_bound
        mul_nonneg(c.abs, base_bound)
        c.abs * base_bound >= Real.0
        Real.1.is_positive
        lt_add_pos(c.abs * base_bound, Real.1)
        c.abs * base_bound < bound
        lte_lt_trans(Real.0, c.abs * base_bound, bound)
        Real.0 < bound
        gt_zero_imp_pos(bound)
        bound.is_positive
        forall(n: Nat) {
            mul_seq(c, a, n) = c * a(n)
            mul_seq(c, a)(n).abs = (c * a(n)).abs
            mul_abs(c, a(n))
            (c * a(n)).abs = c.abs * a(n).abs
            a(n).abs < base_bound
            lt_imp_lte[Real](a(n).abs, base_bound)
            a(n).abs <= base_bound
            not c.abs.is_negative
            lte_mul_nonneg_left(a(n).abs, base_bound, c.abs)
            c.abs * a(n).abs <= c.abs * base_bound
            Real.1.is_positive
            lt_add_pos(c.abs * base_bound, Real.1)
            c.abs * base_bound < bound
            lte_lt_trans(c.abs * a(n).abs, c.abs * base_bound, bound)
            c.abs * a(n).abs < bound
            mul_seq(c, a)(n).abs < bound
        }
    }
}

/// Addition preserves boundedness.
theorem bounded_add_seq(a: Nat -> Real, b: Nat -> Real) {
    is_bounded_seq(a) and is_bounded_seq(b) implies is_bounded_seq(add_seq(a, b))
} by {
    if is_bounded_seq(a) and is_bounded_seq(b) {
        bounded_seq_positive_bound(a)
        bounded_seq_positive_bound(b)
        let a_bound: Real satisfy {
            a_bound.is_positive and forall(n: Nat) {
                a(n).abs < a_bound
            }
        }
        let b_bound: Real satisfy {
            b_bound.is_positive and forall(n: Nat) {
                b(n).abs < b_bound
            }
        }
        let bound = a_bound + b_bound
        forall(n: Nat) {
            add_seq(a, b, n) = a(n) + b(n)
            triangle_ineq(a(n), b(n))
            (a(n) + b(n)).abs <= a(n).abs + b(n).abs
            a(n).abs < a_bound
            b(n).abs < b_bound
            add_lt_lt(a(n).abs, a_bound, b(n).abs, b_bound)
            a(n).abs + b(n).abs < a_bound + b_bound
            a_bound + b_bound = bound
            lte_lt_trans((a(n) + b(n)).abs, a(n).abs + b(n).abs, bound)
            (a(n) + b(n)).abs < bound
            add_seq(a, b)(n).abs < bound
        }
    }
}

/// Pointwise multiplication preserves boundedness.
theorem bounded_prod_seq(a: Nat -> Real, b: Nat -> Real) {
    is_bounded_seq(a) and is_bounded_seq(b) implies is_bounded_seq(prod_seq(a, b))
} by {
    if is_bounded_seq(a) and is_bounded_seq(b) {
        let a_bound: Real satisfy {
            a_bound.is_positive and forall(n: Nat) {
                a(n).abs < a_bound
            }
        }
        let b_bound: Real satisfy {
            b_bound.is_positive and forall(n: Nat) {
                b(n).abs < b_bound
            }
        }
        let bound = a_bound * b_bound
        forall(n: Nat) {
            prod_seq(a, b, n) = a(n) * b(n)
            mul_abs(a(n), b(n))
            (a(n) * b(n)).abs = a(n).abs * b(n).abs
            a(n).abs < a_bound
            lt_imp_lte(a(n).abs, a_bound)
            b(n).abs < b_bound
            abs_not_neg(a(n))
            abs_not_neg(b(n))
            a_bound.is_positive
            b_bound.is_positive
            mul_le_lt_of_nonneg_pos(a(n).abs, a_bound, b(n).abs, b_bound)
            a(n).abs * b(n).abs < a_bound * b_bound
            prod_seq(a, b)(n).abs < bound
        }
    }
}

/// Termwise negation preserves vanishing.
theorem vanishes_neg_seq(a: Nat -> Real) {
    vanishes(a) implies vanishes(neg_seq(a))
} by {
    if vanishes(a) {
        converges(a)
        limit(a) = Real.0
        neg_seq_converges(a)
        converges(neg_seq(a))
        neg_seq_converges_to(a)
        converges_to(neg_seq(a), -limit(a))
        -limit(a) = Real.0
        converges_to(neg_seq(a), Real.0)
        converges_to(neg_seq(a), limit(neg_seq(a)))
        converges_to_unique(neg_seq(a), Real.0, limit(neg_seq(a)))
        limit(neg_seq(a)) = Real.0
    }
}

/// Scalar multiplication preserves vanishing.
theorem vanishes_mul_seq(c: Real, a: Nat -> Real) {
    vanishes(a) implies vanishes(mul_seq(c, a))
} by {
    if vanishes(a) {
        converges(a)
        limit(a) = Real.0
        converges_to(mul_seq(c, a), c * limit(a))
        c * limit(a) = Real.0
        converges_to(mul_seq(c, a), Real.0)
        converges(mul_seq(c, a))
        converges_to(mul_seq(c, a), limit(mul_seq(c, a)))
        converges_to_unique(mul_seq(c, a), Real.0, limit(mul_seq(c, a)))
        limit(mul_seq(c, a)) = Real.0
    }
}

/// Addition preserves vanishing.
theorem vanishes_add_seq(a: Nat -> Real, b: Nat -> Real) {
    vanishes(a) and vanishes(b) implies vanishes(add_seq(a, b))
} by {
    if vanishes(a) and vanishes(b) {
        converges(a)
        converges(b)
        limit(a) = Real.0
        limit(b) = Real.0
        limit_add_seq(a, b)
        converges_to(add_seq(a, b), limit(a) + limit(b))
        limit(a) + limit(b) = Real.0 + Real.0
        add_zero_right(Real.0)
        Real.0 + Real.0 = Real.0
        limit(a) + limit(b) = Real.0
        converges_to(add_seq(a, b), Real.0)
        add_seq_converges(a, b)
        converges(add_seq(a, b))
        converges_imp_converges_to(add_seq(a, b))
        converges_to(add_seq(a, b), limit(add_seq(a, b)))
        converges_to_unique(add_seq(a, b), Real.0, limit(add_seq(a, b)))
        limit(add_seq(a, b)) = Real.0
        vanishes(add_seq(a, b))
    }
}

/// Multiplying a convergent sequence by a vanishing sequence gives a vanishing sequence.
theorem convergent_mul_vanishing_seq(a: Nat -> Real, b: Nat -> Real) {
    converges(a) and vanishes(b) implies vanishes(prod_seq(a, b))
} by {
    if converges(a) and vanishes(b) {
        vanishes_prod_seq(a, b)
        vanishes(prod_seq(a, b))
    }
}

/// Multiplying a vanishing sequence by a convergent sequence gives a vanishing sequence.
theorem vanishing_mul_convergent_seq(a: Nat -> Real, b: Nat -> Real) {
    vanishes(a) and converges(b) implies vanishes(prod_seq(a, b))
} by {
    if vanishes(a) and converges(b) {
        vanishes_prod_seq(b, a)
        forall(n: Nat) {
            prod_seq(a, b, n) = a(n) * b(n)
            prod_seq(b, a, n) = b(n) * a(n)
            real_mul_comm(a(n), b(n))
            prod_seq(a, b, n) = prod_seq(b, a, n)
        }
        prod_seq(a, b) = prod_seq(b, a)
        vanishes(prod_seq(b, a))
        vanishes(prod_seq(a, b))
    }
}

/// The pointwise product is symmetric.
theorem prod_seq_comm(a: Nat -> Real, b: Nat -> Real) {
    prod_seq(a, b) = prod_seq(b, a)
} by {
    forall(n: Nat) {
        prod_seq(a, b, n) = a(n) * b(n)
        prod_seq(b, a, n) = b(n) * a(n)
        a(n) * b(n) = b(n) * a(n)
        prod_seq(a, b, n) = prod_seq(b, a, n)
    }
}

/// A sequence with a convergence tail bound for every positive tolerance converges to that value.
theorem tail_bounds_imp_converges_to(q: Nat -> Real, x: Real) {
    (forall(eps: Real) {
        eps.is_positive implies exists(n: Nat) {
            tail_bound(q, x, n, eps)
        }
    }) implies converges_to(q, x)
} by {
    if forall(eps: Real) {
        eps.is_positive implies exists(n: Nat) {
            tail_bound(q, x, n, eps)
        }
    } {
        converges_to(q, x) = forall(eps: Real) {
            eps.is_positive implies exists(n: Nat) {
                tail_bound(q, x, n, eps)
            }
        }
    }
}

/// A real sequence converging to zero is vanishing.
theorem converges_to_zero_imp_vanishes(q: Nat -> Real) {
    converges_to(q, Real.0) implies vanishes(q)
} by {
    if converges_to(q, Real.0) {
        converges(q)
        converges_to(q, limit(q))
        converges_to_unique(q, Real.0, limit(q))
        limit(q) = Real.0
        vanishes(q)
    }
}

/// Multiplying a bounded sequence by a vanishing sequence gives a convergence tail bound.
theorem bounded_mul_vanishing_seq_tail_bound(a: Nat -> Real, b: Nat -> Real) {
    is_bounded_seq(a) and vanishes(b) implies forall(eps: Real) {
        eps.is_positive implies exists(n: Nat) {
            tail_bound(prod_seq(a, b), Real.0, n, eps)
        }
    }
} by {
    if is_bounded_seq(a) and vanishes(b) {
        bounded_seq_positive_bound(a)
        let bound: Real satisfy {
            bound.is_positive and forall(n: Nat) {
                a(n).abs < bound
            }
        }
        converges(b)
        converges_imp_converges_to(b)
        converges_to(b, limit(b))
        limit(b) = Real.0
        converges_to(b, Real.0)
        converges_to(b, Real.0) = forall(eps: Real) {
            eps.is_positive implies exists(n: Nat) {
                tail_bound(b, Real.0, n, eps)
            }
        }
        forall(eps: Real) {
            if eps.is_positive {
                exists_small_mul_variant_2(bound, eps)
                let delta: Real satisfy {
                    delta.is_positive and delta * bound < eps
                }
                delta * bound = bound * delta
                bound * delta < eps
                delta.is_positive implies exists(n: Nat) {
                    tail_bound(b, Real.0, n, delta)
                }
                let n: Nat satisfy {
                    tail_bound(b, Real.0, n, delta)
                }
                forall(i: Nat) {
                    if n <= i {
                        tail_bound_implies_is_close(b, Real.0, n, delta, i)
                        b(i).is_close(Real.0, delta)
                        (b(i) - Real.0).abs < delta
                        b(i) - Real.0 = b(i)
                        b(i).abs < delta
                        prod_seq(a, b, i) = a(i) * b(i)
                        a(i).abs < bound
                        lt_imp_lte(a(i).abs, bound)
                        a(i).abs <= bound
                        not a(i).abs.is_negative
                        not b(i).abs.is_negative
                        mul_le_lt_of_nonneg_pos(a(i).abs, bound, b(i).abs, delta)
                        a(i).abs * b(i).abs < bound * delta
                        lt_imp_lte(a(i).abs * b(i).abs, bound * delta)
                        bound * delta < eps
                        lte_lt_trans(a(i).abs * b(i).abs, bound * delta, eps)
                        a(i).abs * b(i).abs < eps
                        mul_abs(a(i), b(i))
                        (a(i) * b(i)).abs = a(i).abs * b(i).abs
                        (a(i) * b(i)).abs < eps
                        prod_seq(a, b)(i).abs < eps
                        prod_seq(a, b)(i).is_close(Real.0, eps)
                    }
                }
                tail_bound(prod_seq(a, b), Real.0, n, eps)
            }
        }
    }
}

/// Multiplying a bounded sequence by a vanishing sequence gives a vanishing sequence.
theorem bounded_mul_vanishing_seq(a: Nat -> Real, b: Nat -> Real) {
    is_bounded_seq(a) and vanishes(b) implies vanishes(prod_seq(a, b))
} by {
    if is_bounded_seq(a) and vanishes(b) {
        bounded_mul_vanishing_seq_tail_bound(a, b)
        tail_bounds_imp_converges_to(prod_seq(a, b), Real.0)
        converges_to_zero_imp_vanishes(prod_seq(a, b))
        vanishes(prod_seq(a, b))
    }
}

/// Multiplying a vanishing sequence by a bounded sequence gives a vanishing sequence.
theorem vanishing_mul_bounded_seq(a: Nat -> Real, b: Nat -> Real) {
    vanishes(a) and is_bounded_seq(b) implies vanishes(prod_seq(a, b))
} by {
    if vanishes(a) and is_bounded_seq(b) {
        bounded_mul_vanishing_seq(b, a)
        vanishes(prod_seq(b, a))
        prod_seq_comm(a, b)
        prod_seq(a, b) = prod_seq(b, a)
        vanishes(prod_seq(a, b))
    }
}
