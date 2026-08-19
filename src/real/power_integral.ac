/// Power functions on the reals: derivatives and integrals over the unit
/// interval.
///
/// The main results are:
///   - derivative_pow_real_suc:  the derivative of t ↦ t^(n+1) is
///     t ↦ (n+1)·t^n,
///   - monomial_lipschitz_unit:  the power function t ↦ t^n is n-Lipschitz
///     on [0, 1],
///   - monomial_integrable_unit: t ↦ t^n is integrable on [0, 1],
///   - integral_unit_pow:        the integral of t ↦ t^n over [0, 1] is
///     1/(n+1).
///
/// All value computations go through the fundamental theorem of calculus
/// (ftc2_general from integral_exp.ac): the integrand t ↦ t^n is paired with
/// the antiderivative t ↦ t^(n+1)/(n+1), and integrability follows from the
/// Lipschitz criterion fn_integrable_gen from integral_trig.ac.

from nat import Nat, from_nat, alt_induction, from_nat_one, from_nat_add
from order import lte_trans, lt_imp_lte
from data.basic.functions import function_extensionality, identity_fn, function_eq_transport_predicate_rev
from data.basic.function_algebra import pointwise_mul, pointwise_add
from data.basic.logic import eq_true_intro
from real.real_field import Real
from real.continuity_pow import pow_real_fn, continuous_pow_real_fn, pow_real_fn_suc, pow_real_fn_zero
from real.continuity_base import continuous
from real.continuity_const_mul import continuous_const_mul_left
from real.calculus_api import is_derivative_fn, derivative_fn_identity, derivative_fn_const_mul, derivative_fn_mul
from real.exp import pow_suc, one_pow, zero_pow_pos, mul_one_over, mul_frac_assoc, from_nat_real_nonneg
from real.harmonic import from_nat_suc_pos_real
from real.integral import integral, is_integrable, interval_contains, interval_contains_left, interval_contains_right
from real.integral_exp import ftc2_general
from real.integral_trig import fn_integrable_gen
from real.derivative_continuity import div_mul_cancel_denominator
from real.real_ring import mul_assoc, real_mul_comm, mul_abs, mul_distrib_left
from real.real_series import pow_nonneg, abs_pow, triangle_ineq
from real.derivative_trig import abs_of_nonneg
from real.real_base import add_comm, add_assoc, lte_abs, abs_neg, abs_gte_zero
from ordered_field import mul_le_mul_of_nonneg_right, zero_is_smaller_than_one
from algebra.add_ordered_group import add_le_add_right, add_le_add

numerals Real
numerals Nat

// ---------------------------------------------------------------------------
// The derivative of the power functions
// ---------------------------------------------------------------------------

/// The derivative of t ↦ t^(n+1), namely t ↦ (n+1)·t^n.
define pow_deriv_suc(n: Nat, x: Real) -> Real {
    from_nat[Real](n.suc) * pow_real_fn(n, x)
}

/// The power function t ↦ t^1 is the identity.
lemma pow_real_fn_one_is_identity {
    pow_real_fn(Nat.1) = identity_fn[Real]
} by {
    pow_real_fn_suc(Nat.0)
    pow_real_fn(Nat.0.suc) = pointwise_mul[Real, Real](identity_fn[Real], pow_real_fn(Nat.0))
    pow_real_fn_zero
    forall(x: Real) {
        pointwise_mul[Real, Real](identity_fn[Real], pow_real_fn(Nat.0), x) =
            identity_fn[Real](x) * pow_real_fn(Nat.0, x)
        pow_real_fn(Nat.0, x) = x.pow(Nat.0)
        x.pow(Nat.0) = Real.1
        identity_fn[Real](x) = x
        x * Real.1 = x
        pointwise_mul[Real, Real](identity_fn[Real], pow_real_fn(Nat.0), x) = x
        identity_fn[Real](x) = x
        pointwise_mul[Real, Real](identity_fn[Real], pow_real_fn(Nat.0), x) = identity_fn[Real](x)
    }
    function_extensionality(pointwise_mul[Real, Real](identity_fn[Real], pow_real_fn(Nat.0)), identity_fn[Real])
    pow_real_fn(Nat.0.suc) = identity_fn[Real]
}

/// The zeroth derivative-of-successor function is the constant one function.
lemma pow_deriv_suc_zero_is_one {
    pow_deriv_suc(Nat.0) = constant[Real, Real](Real.1)
} by {
    forall(x: Real) {
        pow_deriv_suc(Nat.0, x) = from_nat[Real](Nat.0.suc) * pow_real_fn(Nat.0, x)
        from_nat_one[Real]
        from_nat[Real](Nat.1) = Real.1
        from_nat[Real](Nat.0.suc) = Real.1
        pow_real_fn(Nat.0, x) = x.pow(Nat.0)
        x.pow(Nat.0) = Real.1
        Real.1 * Real.1 = Real.1
        constant[Real, Real](Real.1, x) = Real.1
        pow_deriv_suc(Nat.0, x) = constant[Real, Real](Real.1, x)
    }
    function_extensionality(pow_deriv_suc(Nat.0), constant[Real, Real](Real.1))
}

/// Pointwise identity for the inductive step: (n+2)·t^(n+1) is the sum of
/// t·(n+1)·t^n and t^(n+1)·1.
lemma pow_deriv_suc_step_pointwise(n: Nat) {
    forall(x: Real) {
        pow_deriv_suc(n.suc, x) =
            pointwise_add(
                pointwise_mul[Real, Real](identity_fn[Real], pow_deriv_suc(n)),
                pointwise_mul[Real, Real](pow_real_fn(n.suc), constant[Real, Real](Real.1)), x)
    }
} by {
    forall(x: Real) {
        pointwise_add(
            pointwise_mul[Real, Real](identity_fn[Real], pow_deriv_suc(n)),
            pointwise_mul[Real, Real](pow_real_fn(n.suc), constant[Real, Real](Real.1)), x) =
            pointwise_mul[Real, Real](identity_fn[Real], pow_deriv_suc(n), x) +
            pointwise_mul[Real, Real](pow_real_fn(n.suc), constant[Real, Real](Real.1), x)
        pointwise_mul[Real, Real](identity_fn[Real], pow_deriv_suc(n), x) =
            identity_fn[Real](x) * pow_deriv_suc(n, x)
        identity_fn[Real](x) = x
        pow_deriv_suc(n, x) = from_nat[Real](n.suc) * pow_real_fn(n, x)
        pointwise_mul[Real, Real](pow_real_fn(n.suc), constant[Real, Real](Real.1), x) =
            pow_real_fn(n.suc, x) * constant[Real, Real](Real.1, x)
        pow_real_fn(n.suc, x) = x.pow(n.suc)
        constant[Real, Real](Real.1, x) = Real.1
        pow_deriv_suc(n.suc, x) = from_nat[Real](n.suc.suc) * pow_real_fn(n.suc, x)
        from_nat_add[Real](n.suc, Nat.1)
        from_nat[Real](n.suc + Nat.1) = from_nat[Real](n.suc) + from_nat[Real](Nat.1)
        n.suc + Nat.1 = n.suc.suc
        from_nat[Real](n.suc.suc) = from_nat[Real](n.suc) + from_nat[Real](Nat.1)
        from_nat_one[Real]
        from_nat[Real](Nat.1) = Real.1
        pow_deriv_suc(n.suc, x) = (from_nat[Real](n.suc) + Real.1) * pow_real_fn(n.suc, x)
        pow_deriv_suc(n.suc, x) = from_nat[Real](n.suc) * pow_real_fn(n.suc, x) + Real.1 * pow_real_fn(n.suc, x)
        from_nat[Real](n.suc) * pow_real_fn(n.suc, x) = x * pow_deriv_suc(n, x)
        Real.1 * pow_real_fn(n.suc, x) = pow_real_fn(n.suc, x)
        pow_deriv_suc(n.suc, x) = x * pow_deriv_suc(n, x) + pow_real_fn(n.suc, x)
        pow_deriv_suc(n.suc, x) =
            pointwise_add(
                pointwise_mul[Real, Real](identity_fn[Real], pow_deriv_suc(n)),
                pointwise_mul[Real, Real](pow_real_fn(n.suc), constant[Real, Real](Real.1)), x)
    }
}

/// The derivative of the power function t ↦ t^(n+1) is t ↦ (n+1)·t^n.
theorem derivative_pow_real_suc(n: Nat) {
    is_derivative_fn(pow_real_fn(n.suc), pow_deriv_suc(n))
} by {
    define p(x: Nat) -> Bool {
        is_derivative_fn(pow_real_fn(x.suc), pow_deriv_suc(x))
    }

    // Base case: the derivative of the identity is one, and the identity is
    // the first power function.
    pow_real_fn_one_is_identity
    pow_real_fn(Nat.1) = identity_fn[Real]
    derivative_fn_identity
    is_derivative_fn(identity_fn[Real], constant[Real, Real](Real.1))
    pow_deriv_suc_zero_is_one
    pow_deriv_suc(Nat.0) = constant[Real, Real](Real.1)
    function_eq_transport_predicate_rev[Real, Real](
        function(h: Real -> Real) { is_derivative_fn(pow_real_fn(Nat.1), h) },
        pow_deriv_suc(Nat.0), constant[Real, Real](Real.1))
    is_derivative_fn(pow_real_fn(Nat.1), pow_deriv_suc(Nat.0))
    p(Nat.0)

    // Inductive step: t^(n+2) = t · t^(n+1), so its derivative is the
    // product rule applied to the identity and t^(n+1).
    forall(x: Nat) {
        if p(x) {
            is_derivative_fn(pow_real_fn(x.suc), pow_deriv_suc(x))
            derivative_fn_identity
            is_derivative_fn(identity_fn[Real], constant[Real, Real](Real.1))
            derivative_fn_mul(identity_fn[Real], pow_real_fn(x.suc),
                constant[Real, Real](Real.1), pow_deriv_suc(x))
            is_derivative_fn(
                pointwise_mul[Real, Real](identity_fn[Real], pow_real_fn(x.suc)),
                pointwise_add(
                    pointwise_mul[Real, Real](identity_fn[Real], pow_deriv_suc(x)),
                    pointwise_mul[Real, Real](pow_real_fn(x.suc), constant[Real, Real](Real.1))))
            pow_real_fn_suc(x.suc)
            pow_real_fn(x.suc.suc) =
                pointwise_mul[Real, Real](identity_fn[Real], pow_real_fn(x.suc))
            function_eq_transport_predicate_rev[Real, Real](
                function(h: Real -> Real) {
                    is_derivative_fn(h,
                        pointwise_add(
                            pointwise_mul[Real, Real](identity_fn[Real], pow_deriv_suc(x)),
                            pointwise_mul[Real, Real](pow_real_fn(x.suc), constant[Real, Real](Real.1))))
                },
                pow_real_fn(x.suc.suc),
                pointwise_mul[Real, Real](identity_fn[Real], pow_real_fn(x.suc)))
            is_derivative_fn(pow_real_fn(x.suc.suc),
                pointwise_add(
                    pointwise_mul[Real, Real](identity_fn[Real], pow_deriv_suc(x)),
                    pointwise_mul[Real, Real](pow_real_fn(x.suc), constant[Real, Real](Real.1))))
            pow_deriv_suc_step_pointwise(x)
            pow_deriv_suc(x.suc) =
                pointwise_add(
                    pointwise_mul[Real, Real](identity_fn[Real], pow_deriv_suc(x)),
                    pointwise_mul[Real, Real](pow_real_fn(x.suc), constant[Real, Real](Real.1)))
            function_eq_transport_predicate_rev[Real, Real](
                function(h: Real -> Real) {
                    is_derivative_fn(pow_real_fn(x.suc.suc), h)
                },
                pow_deriv_suc(x.suc),
                pointwise_add(
                    pointwise_mul[Real, Real](identity_fn[Real], pow_deriv_suc(x)),
                    pointwise_mul[Real, Real](pow_real_fn(x.suc), constant[Real, Real](Real.1))))
            is_derivative_fn(pow_real_fn(x.suc.suc), pow_deriv_suc(x.suc))
            p(x.suc)
        }
    }

    p(Nat.0) and forall(x: Nat) { p(x) implies p(x.suc) }
    alt_induction(p)
    forall(x: Nat) { p(x) }
    p(n)
}


// ---------------------------------------------------------------------------
// The unit-interval antiderivative of the power functions
// ---------------------------------------------------------------------------

/// The antiderivative t ↦ t^(n+1)/(n+1) of the power function t ↦ t^n.
define unit_pow_anti(n: Nat, x: Real) -> Real {
    pointwise_mul[Real, Real](
        constant[Real, Real](Real.1 / from_nat[Real](n.suc)),
        pow_real_fn(n.suc))(x)
}

/// The unit antiderivative is continuous.
lemma unit_pow_anti_continuous(n: Nat) {
    continuous(pointwise_mul[Real, Real](
        constant[Real, Real](Real.1 / from_nat[Real](n.suc)),
        pow_real_fn(n.suc)))
} by {
    continuous_pow_real_fn(n.suc)
    continuous(pow_real_fn(n.suc))
    continuous_const_mul_left(Real.1 / from_nat[Real](n.suc), pow_real_fn(n.suc))
    continuous(pointwise_mul[Real, Real](
        constant[Real, Real](Real.1 / from_nat[Real](n.suc)),
        pow_real_fn(n.suc)))
}

/// Pointwise: (1/(n+1))·(n+1)·t^n equals t^n.
lemma unit_pow_anti_const_mul_deriv(n: Nat) {
    forall(x: Real) {
        pointwise_mul[Real, Real](
            constant[Real, Real](Real.1 / from_nat[Real](n.suc)),
            pow_deriv_suc(n), x) = pow_real_fn(n, x)
    }
} by {
    forall(x: Real) {
        pointwise_mul[Real, Real](
            constant[Real, Real](Real.1 / from_nat[Real](n.suc)),
            pow_deriv_suc(n), x) =
            constant[Real, Real](Real.1 / from_nat[Real](n.suc), x) * pow_deriv_suc(n, x)
        constant[Real, Real](Real.1 / from_nat[Real](n.suc), x) = Real.1 / from_nat[Real](n.suc)
        pow_deriv_suc(n, x) = from_nat[Real](n.suc) * pow_real_fn(n, x)
        (Real.1 / from_nat[Real](n.suc)) * (from_nat[Real](n.suc) * pow_real_fn(n, x)) =
            ((Real.1 / from_nat[Real](n.suc)) * from_nat[Real](n.suc)) * pow_real_fn(n, x)
        from_nat_suc_pos_real(n)
        from_nat[Real](n.suc) > Real.0
        from_nat[Real](n.suc) != Real.0
        div_mul_cancel_denominator(Real.1, from_nat[Real](n.suc))
        (Real.1 / from_nat[Real](n.suc)) * from_nat[Real](n.suc) = Real.1
        ((Real.1 / from_nat[Real](n.suc)) * from_nat[Real](n.suc)) * pow_real_fn(n, x) =
            Real.1 * pow_real_fn(n, x)
        Real.1 * pow_real_fn(n, x) = pow_real_fn(n, x)
        pointwise_mul[Real, Real](
            constant[Real, Real](Real.1 / from_nat[Real](n.suc)),
            pow_deriv_suc(n), x) = pow_real_fn(n, x)
    }
    function_extensionality(pointwise_mul[Real, Real](
        constant[Real, Real](Real.1 / from_nat[Real](n.suc)),
        pow_deriv_suc(n)), pow_real_fn(n))
}

/// The derivative of the unit antiderivative is the power function.
theorem unit_pow_anti_derivative(n: Nat) {
    is_derivative_fn(
        pointwise_mul[Real, Real](
            constant[Real, Real](Real.1 / from_nat[Real](n.suc)),
            pow_real_fn(n.suc)),
        pow_real_fn(n))
} by {
    derivative_pow_real_suc(n)
    is_derivative_fn(pow_real_fn(n.suc), pow_deriv_suc(n))
    derivative_fn_const_mul(Real.1 / from_nat[Real](n.suc), pow_real_fn(n.suc), pow_deriv_suc(n))
    is_derivative_fn(
        pointwise_mul[Real, Real](
            constant[Real, Real](Real.1 / from_nat[Real](n.suc)),
            pow_real_fn(n.suc)),
        pointwise_mul[Real, Real](
            constant[Real, Real](Real.1 / from_nat[Real](n.suc)),
            pow_deriv_suc(n)))
    unit_pow_anti_const_mul_deriv(n)
    pointwise_mul[Real, Real](
        constant[Real, Real](Real.1 / from_nat[Real](n.suc)),
        pow_deriv_suc(n)) = pow_real_fn(n)
    function_eq_transport_predicate_rev[Real, Real](
        function(h: Real -> Real) {
            is_derivative_fn(
                pointwise_mul[Real, Real](
                    constant[Real, Real](Real.1 / from_nat[Real](n.suc)),
                    pow_real_fn(n.suc)),
                h)
        },
        pointwise_mul[Real, Real](
            constant[Real, Real](Real.1 / from_nat[Real](n.suc)),
            pow_deriv_suc(n)),
        pow_real_fn(n))
    is_derivative_fn(
        pointwise_mul[Real, Real](
            constant[Real, Real](Real.1 / from_nat[Real](n.suc)),
            pow_real_fn(n.suc)),
        pow_real_fn(n))
}


// ---------------------------------------------------------------------------
// Bounds and Lipschitz constants for power functions on [0, 1]
// ---------------------------------------------------------------------------

/// The power function is nonnegative on [0, 1].
theorem pow_real_fn_nonneg_unit(n: Nat) {
    forall(t: Real) {
        interval_contains(Real.0, Real.1, t) implies Real.0 <= pow_real_fn(n, t)
    }
} by {
    forall(t: Real) {
        if interval_contains(Real.0, Real.1, t) {
            interval_contains_left(Real.0, Real.1, t)
            Real.0 <= t
            pow_nonneg(t, n)
            Real.0 <= t.pow(n)
            pow_real_fn(n, t) = t.pow(n)
            Real.0 <= pow_real_fn(n, t)
        }
    }
}

/// The power function is at most one on [0, 1].
theorem pow_real_fn_le_one_unit(n: Nat) {
    forall(t: Real) {
        interval_contains(Real.0, Real.1, t) implies pow_real_fn(n, t) <= Real.1
    }
} by {
    define p(x: Nat) -> Bool {
        forall(t: Real) {
            interval_contains(Real.0, Real.1, t) implies pow_real_fn(x, t) <= Real.1
        }
    }

    // Base case: t^0 = 1.
    forall(t: Real) {
        if interval_contains(Real.0, Real.1, t) {
            pow_real_fn(Nat.0, t) = t.pow(Nat.0)
            t.pow(Nat.0) = Real.1
            pow_real_fn(Nat.0, t) <= Real.1
        }
    }
    p(Nat.0)

    // Inductive step: t^(n+1) = t·t^n <= t <= 1.
    forall(x: Nat) {
        if p(x) {
            forall(t: Real) {
                if interval_contains(Real.0, Real.1, t) {
                    interval_contains_left(Real.0, Real.1, t)
                    Real.0 <= t
                    p(x)
                    pow_real_fn(x, t) <= Real.1
                    mul_le_mul_of_nonneg_right[Real](pow_real_fn(x, t), Real.1, t)
                    pow_real_fn(x, t) * t <= Real.1 * t
                    pow_real_fn(x, t) * t = t * pow_real_fn(x, t)
                    Real.1 * t = t
                    t * pow_real_fn(x, t) <= t
                    interval_contains_right(Real.0, Real.1, t)
                    t <= Real.1
                    lte_trans[Real](t * pow_real_fn(x, t), t, Real.1)
                    t * pow_real_fn(x, t) <= Real.1
                    pow_real_fn(x.suc, t) = t.pow(x.suc)
                    pow_suc(t, x)
                    t.pow(x.suc) = t * t.pow(x)
                    t * pow_real_fn(x, t) = t * t.pow(x)
                    pow_real_fn(x.suc, t) <= Real.1
                }
            }
            p(x.suc)
        }
    }

    p(Nat.0) and forall(x: Nat) { p(x) implies p(x.suc) }
    alt_induction(p)
    forall(x: Nat) { p(x) }
    p(n)
}

/// The absolute difference of the successor powers factors through the
/// difference of the lower powers.
lemma monomial_lipschitz_step_bound(n: Nat, u: Real, v: Real) {
    (u * pow_real_fn(n, u) - v * pow_real_fn(n, v)).abs <= u.abs * (pow_real_fn(n, u) - pow_real_fn(n, v)).abs + pow_real_fn(n, v).abs * (u - v).abs
} by {
    u * pow_real_fn(n, u) - v * pow_real_fn(n, v) =
        u * (pow_real_fn(n, u) - pow_real_fn(n, v)) + pow_real_fn(n, v) * (u - v)
    triangle_ineq(u * (pow_real_fn(n, u) - pow_real_fn(n, v)), pow_real_fn(n, v) * (u - v))
    (u * (pow_real_fn(n, u) - pow_real_fn(n, v)) + pow_real_fn(n, v) * (u - v)).abs <= (u * (pow_real_fn(n, u) - pow_real_fn(n, v))).abs + (pow_real_fn(n, v) * (u - v)).abs
    mul_abs(u, pow_real_fn(n, u) - pow_real_fn(n, v))
    (u * (pow_real_fn(n, u) - pow_real_fn(n, v))).abs = u.abs * (pow_real_fn(n, u) - pow_real_fn(n, v)).abs
    mul_abs(pow_real_fn(n, v), u - v)
    (pow_real_fn(n, v) * (u - v)).abs = pow_real_fn(n, v).abs * (u - v).abs
    (u * pow_real_fn(n, u) - v * pow_real_fn(n, v)).abs <= u.abs * (pow_real_fn(n, u) - pow_real_fn(n, v)).abs + pow_real_fn(n, v).abs * (u - v).abs
}

/// The power function is n-Lipschitz on [0, 1].
theorem monomial_lipschitz_unit(n: Nat) {
    forall(u: Real, v: Real) {
        interval_contains(Real.0, Real.1, u) and interval_contains(Real.0, Real.1, v) implies
        (pow_real_fn(n, u) - pow_real_fn(n, v)).abs <= from_nat[Real](n) * (u - v).abs
    }
} by {
    define p(x: Nat) -> Bool {
        forall(u: Real, v: Real) {
            interval_contains(Real.0, Real.1, u) and interval_contains(Real.0, Real.1, v) implies
            (pow_real_fn(x, u) - pow_real_fn(x, v)).abs <= from_nat[Real](x) * (u - v).abs
        }
    }

    // Base case: t^0 is constant, so the difference is zero.
    forall(u: Real, v: Real) {
        if interval_contains(Real.0, Real.1, u) and interval_contains(Real.0, Real.1, v) {
            pow_real_fn(Nat.0, u) = u.pow(Nat.0)
            u.pow(Nat.0) = Real.1
            pow_real_fn(Nat.0, v) = v.pow(Nat.0)
            v.pow(Nat.0) = Real.1
            pow_real_fn(Nat.0, u) - pow_real_fn(Nat.0, v) = Real.1 - Real.1
            Real.1 - Real.1 = Real.0
            pow_real_fn(Nat.0, u) - pow_real_fn(Nat.0, v) = Real.0
            (pow_real_fn(Nat.0, u) - pow_real_fn(Nat.0, v)).abs = Real.0.abs
            Real.0.abs = Real.0
            from_nat[Real](Nat.0) * (u - v).abs = Real.0 * (u - v).abs
            Real.0 * (u - v).abs = Real.0
            (pow_real_fn(Nat.0, u) - pow_real_fn(Nat.0, v)).abs <= from_nat[Real](Nat.0) * (u - v).abs
        }
    }
    p(Nat.0)

    // Inductive step: |u^(n+1) - v^(n+1)| <= |u|·|u^n - v^n| + |v^n|·|u - v|.
    forall(x: Nat) {
        if p(x) {
            forall(u: Real, v: Real) {
                if interval_contains(Real.0, Real.1, u) and interval_contains(Real.0, Real.1, v) {
                    p(x)
                    (pow_real_fn(x, u) - pow_real_fn(x, v)).abs <= from_nat[Real](x) * (u - v).abs
                    monomial_lipschitz_step_bound(x, u, v)
                    (u * pow_real_fn(x, u) - v * pow_real_fn(x, v)).abs <= u.abs * (pow_real_fn(x, u) - pow_real_fn(x, v)).abs + pow_real_fn(x, v).abs * (u - v).abs
                    interval_contains_left(Real.0, Real.1, u)
                    Real.0 <= u
                    abs_of_nonneg(u)
                    u.abs = u
                    interval_contains_right(Real.0, Real.1, u)
                    u <= Real.1
                    interval_contains_left(Real.0, Real.1, v)
                    Real.0 <= v
                    pow_nonneg(v, x)
                    Real.0 <= v.pow(x)
                    pow_real_fn(x, v) = v.pow(x)
                    Real.0 <= pow_real_fn(x, v)
                    abs_of_nonneg(pow_real_fn(x, v))
                    pow_real_fn(x, v).abs = pow_real_fn(x, v)
                    pow_real_fn_le_one_unit(x)
                    pow_real_fn(x, v) <= Real.1
                    abs_gte_zero(u - v)
                    Real.0 <= (u - v).abs
                    abs_gte_zero(pow_real_fn(x, u) - pow_real_fn(x, v))
                    Real.0 <= (pow_real_fn(x, u) - pow_real_fn(x, v)).abs
                    mul_le_mul_of_nonneg_right[Real](u.abs, Real.1, (pow_real_fn(x, u) - pow_real_fn(x, v)).abs)
                    u.abs * (pow_real_fn(x, u) - pow_real_fn(x, v)).abs <= Real.1 * (pow_real_fn(x, u) - pow_real_fn(x, v)).abs
                    Real.1 * (pow_real_fn(x, u) - pow_real_fn(x, v)).abs = (pow_real_fn(x, u) - pow_real_fn(x, v)).abs
                    lte_trans[Real](u.abs * (pow_real_fn(x, u) - pow_real_fn(x, v)).abs, (pow_real_fn(x, u) - pow_real_fn(x, v)).abs, from_nat[Real](x) * (u - v).abs)
                    u.abs * (pow_real_fn(x, u) - pow_real_fn(x, v)).abs <= from_nat[Real](x) * (u - v).abs
                    mul_le_mul_of_nonneg_right[Real](pow_real_fn(x, v), Real.1, (u - v).abs)
                    pow_real_fn(x, v) * (u - v).abs <= Real.1 * (u - v).abs
                    Real.1 * (u - v).abs = (u - v).abs
                    pow_real_fn(x, v) * (u - v).abs = pow_real_fn(x, v).abs * (u - v).abs
                    pow_real_fn(x, v).abs * (u - v).abs <= (u - v).abs
                    add_le_add(
                        u.abs * (pow_real_fn(x, u) - pow_real_fn(x, v)).abs,
                        from_nat[Real](x) * (u - v).abs,
                        pow_real_fn(x, v).abs * (u - v).abs,
                        (u - v).abs)
                    u.abs * (pow_real_fn(x, u) - pow_real_fn(x, v)).abs +
                        pow_real_fn(x, v).abs * (u - v).abs <= from_nat[Real](x) * (u - v).abs + (u - v).abs
                    Real.1 * (u - v).abs = (u - v).abs
                    mul_distrib_left(from_nat[Real](x), Real.1, (u - v).abs)
                    (from_nat[Real](x) + Real.1) * (u - v).abs =
                        from_nat[Real](x) * (u - v).abs + Real.1 * (u - v).abs
                    lte_trans[Real](
                        u.abs * (pow_real_fn(x, u) - pow_real_fn(x, v)).abs +
                            pow_real_fn(x, v).abs * (u - v).abs,
                        from_nat[Real](x) * (u - v).abs + (u - v).abs,
                        (from_nat[Real](x) + Real.1) * (u - v).abs)
                    u.abs * (pow_real_fn(x, u) - pow_real_fn(x, v)).abs +
                        pow_real_fn(x, v).abs * (u - v).abs <= (from_nat[Real](x) + Real.1) * (u - v).abs
                    lte_trans[Real](
                        (u * pow_real_fn(x, u) - v * pow_real_fn(x, v)).abs,
                        u.abs * (pow_real_fn(x, u) - pow_real_fn(x, v)).abs +
                            pow_real_fn(x, v).abs * (u - v).abs,
                        (from_nat[Real](x) + Real.1) * (u - v).abs)
                    (u * pow_real_fn(x, u) - v * pow_real_fn(x, v)).abs <= (from_nat[Real](x) + Real.1) * (u - v).abs
                    from_nat_add[Real](x, Nat.1)
                    from_nat[Real](x + Nat.1) = from_nat[Real](x) + from_nat[Real](Nat.1)
                    from_nat_one[Real]
                    from_nat[Real](Nat.1) = Real.1
                    from_nat[Real](x.suc) = from_nat[Real](x) + Real.1
                    x + Nat.1 = x.suc
                    from_nat[Real](x.suc) = from_nat[Real](x) + Real.1
                    pow_real_fn(x.suc, u) = u.pow(x.suc)
                    pow_suc(u, x)
                    u.pow(x.suc) = u * u.pow(x)
                    pow_real_fn(x, u) = u.pow(x)
                    u * pow_real_fn(x, u) = pow_real_fn(x.suc, u)
                    pow_real_fn(x.suc, v) = v.pow(x.suc)
                    pow_suc(v, x)
                    v.pow(x.suc) = v * v.pow(x)
                    pow_real_fn(x, v) = v.pow(x)
                    v * pow_real_fn(x, v) = pow_real_fn(x.suc, v)
                    pow_real_fn(x.suc, u) - pow_real_fn(x.suc, v) =
                        u * pow_real_fn(x, u) - v * pow_real_fn(x, v)
                    (pow_real_fn(x.suc, u) - pow_real_fn(x.suc, v)).abs <= from_nat[Real](x.suc) * (u - v).abs
                }
            }
            p(x.suc)
        }
    }

    p(Nat.0) and forall(x: Nat) { p(x) implies p(x.suc) }
    alt_induction(p)
    forall(x: Nat) { p(x) }
    p(n)
}


// ---------------------------------------------------------------------------
// Integrability and the value of the power integral
// ---------------------------------------------------------------------------

/// The power function is integrable on [0, 1].
theorem monomial_integrable_unit(n: Nat) {
    is_integrable(pow_real_fn(n), Real.0, Real.1)
} by {
    Real.0 <= Real.1
    zero_is_smaller_than_one[Real]
    Real.0 < Real.1
    lt_imp_lte[Real](Real.0, Real.1)
    from_nat_real_nonneg(n)
    from_nat[Real](n) >= Real.0
    Real.0 <= from_nat[Real](n)
    unit_pow_anti_continuous(n)
    continuous(pointwise_mul[Real, Real](
        constant[Real, Real](Real.1 / from_nat[Real](n.suc)),
        pow_real_fn(n.suc)))
    unit_pow_anti_derivative(n)
    is_derivative_fn(
        pointwise_mul[Real, Real](
            constant[Real, Real](Real.1 / from_nat[Real](n.suc)),
            pow_real_fn(n.suc)),
        pow_real_fn(n))
    monomial_lipschitz_unit(n)
    pow_real_fn_nonneg_unit(n)
    pow_real_fn_le_one_unit(n)
    Real.0 <= Real.1
    zero_is_smaller_than_one[Real]
    Real.0 < Real.1
    lt_imp_lte[Real](Real.0, Real.1)
    Real.0 <= from_nat[Real](n)
    eq_true_intro(continuous(pointwise_mul[Real, Real](
        constant[Real, Real](Real.1 / from_nat[Real](n.suc)),
        pow_real_fn(n.suc))))
    (continuous(pointwise_mul[Real, Real](
        constant[Real, Real](Real.1 / from_nat[Real](n.suc)),
        pow_real_fn(n.suc)))) = true
    eq_true_intro(is_derivative_fn(
        pointwise_mul[Real, Real](
            constant[Real, Real](Real.1 / from_nat[Real](n.suc)),
            pow_real_fn(n.suc)),
        pow_real_fn(n)))
    (is_derivative_fn(
        pointwise_mul[Real, Real](
            constant[Real, Real](Real.1 / from_nat[Real](n.suc)),
            pow_real_fn(n.suc)),
        pow_real_fn(n))) = true
    eq_true_intro(forall(u: Real, v: Real) {
        interval_contains(Real.0, Real.1, u) and interval_contains(Real.0, Real.1, v) implies
        (pow_real_fn(n, u) - pow_real_fn(n, v)).abs <= from_nat[Real](n) * (u - v).abs
    })
    (forall(u: Real, v: Real) {
        interval_contains(Real.0, Real.1, u) and interval_contains(Real.0, Real.1, v) implies
        (pow_real_fn(n, u) - pow_real_fn(n, v)).abs <= from_nat[Real](n) * (u - v).abs
    }) = true
    eq_true_intro(forall(t: Real) { interval_contains(Real.0, Real.1, t) implies Real.0 <= pow_real_fn(n, t) })
    (forall(t: Real) { interval_contains(Real.0, Real.1, t) implies Real.0 <= pow_real_fn(n, t) }) = true
    eq_true_intro(forall(t: Real) { interval_contains(Real.0, Real.1, t) implies pow_real_fn(n, t) <= Real.1 })
    (forall(t: Real) { interval_contains(Real.0, Real.1, t) implies pow_real_fn(n, t) <= Real.1 }) = true
    Real.0 <= Real.1 and Real.0 <= from_nat[Real](n) and
        continuous(pointwise_mul[Real, Real](
            constant[Real, Real](Real.1 / from_nat[Real](n.suc)),
            pow_real_fn(n.suc))) and
        is_derivative_fn(
            pointwise_mul[Real, Real](
                constant[Real, Real](Real.1 / from_nat[Real](n.suc)),
                pow_real_fn(n.suc)),
            pow_real_fn(n)) and
        (forall(u: Real, v: Real) {
            interval_contains(Real.0, Real.1, u) and interval_contains(Real.0, Real.1, v) implies
            (pow_real_fn(n, u) - pow_real_fn(n, v)).abs <= from_nat[Real](n) * (u - v).abs
        }) and
        (forall(t: Real) { interval_contains(Real.0, Real.1, t) implies Real.0 <= pow_real_fn(n, t) }) and
        (forall(t: Real) { interval_contains(Real.0, Real.1, t) implies pow_real_fn(n, t) <= Real.1 })
    fn_integrable_gen(pow_real_fn(n),
        pointwise_mul[Real, Real](
            constant[Real, Real](Real.1 / from_nat[Real](n.suc)),
            pow_real_fn(n.suc)),
        Real.0, Real.1, from_nat[Real](n), Real.0, Real.1)
    is_integrable(pow_real_fn(n), Real.0, Real.1)
}

/// The unit antiderivative is 1/(n+1) at 1.
lemma unit_pow_anti_value_one(n: Nat) {
    unit_pow_anti(n, Real.1) = Real.1 / from_nat[Real](n.suc)
} by {
    unit_pow_anti(n, Real.1) =
        pointwise_mul[Real, Real](
            constant[Real, Real](Real.1 / from_nat[Real](n.suc)),
            pow_real_fn(n.suc), Real.1)
    pointwise_mul[Real, Real](
        constant[Real, Real](Real.1 / from_nat[Real](n.suc)),
        pow_real_fn(n.suc), Real.1) =
        constant[Real, Real](Real.1 / from_nat[Real](n.suc), Real.1) * pow_real_fn(n.suc, Real.1)
    constant[Real, Real](Real.1 / from_nat[Real](n.suc), Real.1) = Real.1 / from_nat[Real](n.suc)
    pow_real_fn(n.suc, Real.1) = Real.1.pow(n.suc)
    one_pow[Real](n.suc)
    Real.1.pow(n.suc) = Real.1
    (Real.1 / from_nat[Real](n.suc)) * Real.1 = Real.1 / from_nat[Real](n.suc)
    unit_pow_anti(n, Real.1) = Real.1 / from_nat[Real](n.suc)
}


/// The unit antiderivative is zero at 0.
lemma unit_pow_anti_value_zero(n: Nat) {
    unit_pow_anti(n, Real.0) = Real.0
} by {
    unit_pow_anti(n, Real.0) =
        pointwise_mul[Real, Real](
            constant[Real, Real](Real.1 / from_nat[Real](n.suc)),
            pow_real_fn(n.suc), Real.0)
    pointwise_mul[Real, Real](
        constant[Real, Real](Real.1 / from_nat[Real](n.suc)),
        pow_real_fn(n.suc), Real.0) =
        constant[Real, Real](Real.1 / from_nat[Real](n.suc), Real.0) * pow_real_fn(n.suc, Real.0)
    constant[Real, Real](Real.1 / from_nat[Real](n.suc), Real.0) = Real.1 / from_nat[Real](n.suc)
    pow_real_fn(n.suc, Real.0) = Real.0.pow(n.suc)
    n.suc >= Nat.1
    zero_pow_pos(n.suc)
    Real.0.pow(n.suc) = Real.0
    (Real.1 / from_nat[Real](n.suc)) * Real.0 = Real.0
    unit_pow_anti(n, Real.0) = Real.0
}

/// The integral of t^n over [0, 1] is 1/(n+1).
theorem integral_unit_pow(n: Nat) {
    integral(pow_real_fn(n), Real.0, Real.1) = Real.1 / from_nat[Real](n.suc)
} by {
    Real.0 <= Real.1
    zero_is_smaller_than_one[Real]
    Real.0 < Real.1
    lt_imp_lte[Real](Real.0, Real.1)
    unit_pow_anti_continuous(n)
    continuous(pointwise_mul[Real, Real](
        constant[Real, Real](Real.1 / from_nat[Real](n.suc)),
        pow_real_fn(n.suc)))
    unit_pow_anti_derivative(n)
    is_derivative_fn(
        pointwise_mul[Real, Real](
            constant[Real, Real](Real.1 / from_nat[Real](n.suc)),
            pow_real_fn(n.suc)),
        pow_real_fn(n))
    monomial_integrable_unit(n)
    is_integrable(pow_real_fn(n), Real.0, Real.1)
    pow_real_fn_nonneg_unit(n)
    pow_real_fn_le_one_unit(n)
    Real.0 <= Real.1
    zero_is_smaller_than_one[Real]
    Real.0 < Real.1
    lt_imp_lte[Real](Real.0, Real.1)
    eq_true_intro(continuous(pointwise_mul[Real, Real](
        constant[Real, Real](Real.1 / from_nat[Real](n.suc)),
        pow_real_fn(n.suc))))
    (continuous(pointwise_mul[Real, Real](
        constant[Real, Real](Real.1 / from_nat[Real](n.suc)),
        pow_real_fn(n.suc)))) = true
    eq_true_intro(is_derivative_fn(
        pointwise_mul[Real, Real](
            constant[Real, Real](Real.1 / from_nat[Real](n.suc)),
            pow_real_fn(n.suc)),
        pow_real_fn(n)))
    (is_derivative_fn(
        pointwise_mul[Real, Real](
            constant[Real, Real](Real.1 / from_nat[Real](n.suc)),
            pow_real_fn(n.suc)),
        pow_real_fn(n))) = true
    eq_true_intro(forall(t: Real) { interval_contains(Real.0, Real.1, t) implies Real.0 <= pow_real_fn(n, t) })
    (forall(t: Real) { interval_contains(Real.0, Real.1, t) implies Real.0 <= pow_real_fn(n, t) }) = true
    eq_true_intro(forall(t: Real) { interval_contains(Real.0, Real.1, t) implies pow_real_fn(n, t) <= Real.1 })
    (forall(t: Real) { interval_contains(Real.0, Real.1, t) implies pow_real_fn(n, t) <= Real.1 }) = true
    Real.0 <= Real.1 and is_integrable(pow_real_fn(n), Real.0, Real.1) and
        continuous(pointwise_mul[Real, Real](
            constant[Real, Real](Real.1 / from_nat[Real](n.suc)),
            pow_real_fn(n.suc))) and
        is_derivative_fn(
            pointwise_mul[Real, Real](
                constant[Real, Real](Real.1 / from_nat[Real](n.suc)),
                pow_real_fn(n.suc)),
            pow_real_fn(n)) and
        (forall(t: Real) { interval_contains(Real.0, Real.1, t) implies Real.0 <= pow_real_fn(n, t) }) and
        (forall(t: Real) { interval_contains(Real.0, Real.1, t) implies pow_real_fn(n, t) <= Real.1 })
    ftc2_general(pow_real_fn(n),
        pointwise_mul[Real, Real](
            constant[Real, Real](Real.1 / from_nat[Real](n.suc)),
            pow_real_fn(n.suc)),
        Real.0, Real.1, Real.0, Real.1)
    integral(pow_real_fn(n), Real.0, Real.1) =
        pointwise_mul[Real, Real](
            constant[Real, Real](Real.1 / from_nat[Real](n.suc)),
            pow_real_fn(n.suc), Real.1) -
        pointwise_mul[Real, Real](
            constant[Real, Real](Real.1 / from_nat[Real](n.suc)),
            pow_real_fn(n.suc), Real.0)
    unit_pow_anti(n, Real.1) =
        pointwise_mul[Real, Real](
            constant[Real, Real](Real.1 / from_nat[Real](n.suc)),
            pow_real_fn(n.suc), Real.1)
    unit_pow_anti_value_one(n)
    unit_pow_anti(n, Real.1) = Real.1 / from_nat[Real](n.suc)
    unit_pow_anti(n, Real.0) =
        pointwise_mul[Real, Real](
            constant[Real, Real](Real.1 / from_nat[Real](n.suc)),
            pow_real_fn(n.suc), Real.0)
    unit_pow_anti_value_zero(n)
    unit_pow_anti(n, Real.0) = Real.0
    Real.1 / from_nat[Real](n.suc) - Real.0 = Real.1 / from_nat[Real](n.suc)
    integral(pow_real_fn(n), Real.0, Real.1) = Real.1 / from_nat[Real](n.suc)
}
