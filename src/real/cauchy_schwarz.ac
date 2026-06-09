from nat import Nat
from list import partial
from list import partial_pointwise_eq, partial_split_last, partial_zero
from real.real_field import Real
from real.real_base import one_half_plus_one_half, one_half_positive
from real.real_ring import mul_neg_left, mul_neg_one_left, mul_neg_right, mul_nonneg,
    square_nonneg
from real.real_series import is_lower_bound, partial_nonneg, prod_fn
from real.rectangular_sum import add_fn_2, nonneg_fn_2, product_sum, product_sum_factors,
    rectangular_sum, rectangular_sum_add, rectangular_sum_nonneg, rectangular_sum_scale,
    row_sum, scalar_mul_fn_2

numerals Real

/// The pointwise product of two real sequences.
define pointwise_product(a: Nat -> Real, b: Nat -> Real, i: Nat) -> Real {
    a(i) * b(i)
}

/// The pointwise square of a real sequence.
define pointwise_square(a: Nat -> Real, i: Nat) -> Real {
    a(i) * a(i)
}

/// The finite dot product of two real sequences over the first `n` terms.
define finite_dot(a: Nat -> Real, b: Nat -> Real, n: Nat) -> Real {
    partial(pointwise_product(a, b), n)
}

/// The finite sum of squares of a real sequence over the first `n` terms.
define finite_square_sum(a: Nat -> Real, n: Nat) -> Real {
    partial(pointwise_square(a), n)
}

/// Dot products are symmetric over a finite range.
theorem finite_dot_symm(a: Nat -> Real, b: Nat -> Real, n: Nat) {
    finite_dot(a, b, n) = finite_dot(b, a, n)
} by {
    forall(k: Nat) {
        pointwise_product(a, b, k) = a(k) * b(k)
        pointwise_product(b, a, k) = b(k) * a(k)
        b(k) * a(k) = a(k) * b(k)
        pointwise_product(a, b, k) = pointwise_product(b, a, k)
    }
    pointwise_product(a, b) = pointwise_product(b, a)
    finite_dot(a, b, n) = partial(pointwise_product(a, b), n)
    finite_dot(b, a, n) = partial(pointwise_product(b, a), n)
    finite_dot(a, b, n) = finite_dot(b, a, n)
}

/// The dot product over the empty range is zero.
theorem finite_dot_zero(a: Nat -> Real, b: Nat -> Real) {
    finite_dot(a, b, Nat.0) = Real.0
} by {
    partial_zero(pointwise_product(a, b))
    finite_dot(a, b, Nat.0) = partial(pointwise_product(a, b), Nat.0)
}

/// A finite dot product splits off its last term.
theorem finite_dot_suc(a: Nat -> Real, b: Nat -> Real, n: Nat) {
    finite_dot(a, b, n.suc) = finite_dot(a, b, n) + a(n) * b(n)
} by {
    partial_split_last(pointwise_product(a, b), n)
    finite_dot(a, b, n.suc) = partial(pointwise_product(a, b), n.suc)
    partial(pointwise_product(a, b), n.suc) =
        partial(pointwise_product(a, b), n) + pointwise_product(a, b, n)
    finite_dot(a, b, n) = partial(pointwise_product(a, b), n)
    pointwise_product(a, b, n) = a(n) * b(n)
}

/// The square sum over the empty range is zero.
theorem finite_square_sum_zero(a: Nat -> Real) {
    finite_square_sum(a, Nat.0) = Real.0
} by {
    partial_zero(pointwise_square(a))
    finite_square_sum(a, Nat.0) = partial(pointwise_square(a), Nat.0)
}

/// A finite square sum splits off its last square term.
theorem finite_square_sum_suc(a: Nat -> Real, n: Nat) {
    finite_square_sum(a, n.suc) = finite_square_sum(a, n) + a(n) * a(n)
} by {
    partial_split_last(pointwise_square(a), n)
    finite_square_sum(a, n.suc) = partial(pointwise_square(a), n.suc)
    partial(pointwise_square(a), n.suc) =
        partial(pointwise_square(a), n) + pointwise_square(a, n)
    finite_square_sum(a, n) = partial(pointwise_square(a), n)
    pointwise_square(a, n) = a(n) * a(n)
}

/// The square appearing in Lagrange's identity for Cauchy-Schwarz.
define lagrange_square(a: Nat -> Real, b: Nat -> Real, i: Nat, j: Nat) -> Real {
    (a(i) * b(j) - a(j) * b(i)) * (a(i) * b(j) - a(j) * b(i))
}

/// The outer product of two pointwise-square sequences.
define square_outer(a: Nat -> Real, b: Nat -> Real, i: Nat, j: Nat) -> Real {
    pointwise_square(a, i) * pointwise_square(b, j)
}

/// The outer product of the pointwise-product sequence with itself.
define dot_outer(a: Nat -> Real, b: Nat -> Real, i: Nat, j: Nat) -> Real {
    pointwise_product(a, b, i) * pointwise_product(a, b, j)
}

/// The expanded Lagrange square term.
define lagrange_expanded(a: Nat -> Real, b: Nat -> Real, i: Nat, j: Nat) -> Real {
    square_outer(a, b, i, j) + -dot_outer(a, b, i, j) +
        (-dot_outer(a, b, i, j) + square_outer(b, a, i, j))
}

/// The negative of the dot outer product.
define neg_dot_outer(a: Nat -> Real, b: Nat -> Real, i: Nat, j: Nat) -> Real {
    -dot_outer(a, b, i, j)
}

/// The first half of the expanded Lagrange term.
define lagrange_first(a: Nat -> Real, b: Nat -> Real, i: Nat, j: Nat) -> Real {
    square_outer(a, b, i, j) + neg_dot_outer(a, b, i, j)
}

/// The second half of the expanded Lagrange term.
define lagrange_second(a: Nat -> Real, b: Nat -> Real, i: Nat, j: Nat) -> Real {
    neg_dot_outer(a, b, i, j) + square_outer(b, a, i, j)
}

/// The expanded Lagrange term as a sum of two named halves.
define lagrange_pair(a: Nat -> Real, b: Nat -> Real, i: Nat, j: Nat) -> Real {
    lagrange_first(a, b, i, j) + lagrange_second(a, b, i, j)
}

/// Pointwise squares are nonnegative.
theorem pointwise_square_nonnegative(a: Nat -> Real, i: Nat) {
    pointwise_square(a, i) >= Real.0
} by {
    square_nonneg(a(i))
}

/// Finite sums of pointwise squares are nonnegative.
theorem finite_square_sum_nonnegative(a: Nat -> Real, n: Nat) {
    finite_square_sum(a, n) >= Real.0
} by {
    forall(i: Nat) {
        pointwise_square(a, i) >= Real.0
        Real.0 <= pointwise_square(a, i)
    }
    is_lower_bound(pointwise_square(a), Real.0)
    partial_nonneg(pointwise_square(a), n)
}

/// Lagrange square terms are nonnegative.
theorem lagrange_square_nonnegative(a: Nat -> Real, b: Nat -> Real, i: Nat, j: Nat) {
    lagrange_square(a, b, i, j) >= Real.0
} by {
    let x = a(i) * b(j) - a(j) * b(i)
    square_nonneg(x)
}

/// The rectangular sum of Lagrange square terms is nonnegative.
theorem lagrange_square_sum_nonnegative(a: Nat -> Real, b: Nat -> Real, n: Nat) {
    rectangular_sum(lagrange_square(a, b), n, n) >= Real.0
} by {
    forall(i: Nat, j: Nat) {
        lagrange_square_nonnegative(a, b, i, j)
        lagrange_square(a, b, i, j) >= Real.0
    }
    nonneg_fn_2(lagrange_square(a, b))
    rectangular_sum_nonneg(lagrange_square(a, b), n, n)
}

/// The square of a sum, with the two cross terms written separately.
theorem real_square_add_expanded(x: Real, y: Real) {
    (x + y) * (x + y) = x * x + x * y + (x * y + y * y)
} by {
    (x + y) * (x + y) = (x + y) * x + (x + y) * y
    (x + y) * x = x * x + y * x
    (x + y) * y = x * y + y * y
    (x + y) * (x + y) = (x * x + y * x) + (x * y + y * y)
    y * x = x * y
    (x * x + y * x) + (x * y + y * y) = x * x + x * y + (x * y + y * y)
}

/// Products of two products can be regrouped by their left and right factors.
theorem product_pair_square_rearrange(x: Real, y: Real) {
    (x * y) * (x * y) = (x * x) * (y * y)
} by {
    (x * y) * (x * y) = x * (y * x) * y
    y * x = x * y
    x * (x * y) * y = (x * x) * y * y
    (x * x) * y * y = (x * x) * (y * y)
}

/// Products of two cross terms can be regrouped by index.
theorem product_cross_rearrange(w: Real, x: Real, y: Real, z: Real) {
    (w * x) * (y * z) = (w * z) * (y * x)
} by {
    (w * x) * (y * z) = w * x * y * z
    w * x * y * z = ((w * x) * y) * z
    (w * x) * y = w * (x * y)
    x * y = y * x
    w * (x * y) = w * (y * x)
    w * (y * x) = (w * y) * x
    ((w * x) * y) * z = ((w * y) * x) * z
    ((w * y) * x) * z = (w * y) * (x * z)
    x * z = z * x
    (w * y) * (x * z) = (w * y) * (z * x)
    (w * y) * (z * x) = ((w * y) * z) * x
    ((w * y) * z) * x = (w * (y * z)) * x
    y * z = z * y
    w * (y * z) = w * (z * y)
    w * (z * y) = (w * z) * y
    (w * (y * z)) * x = ((w * z) * y) * x
    ((w * z) * y) * x = (w * z) * (y * x)
}

/// The Lagrange square expands into two square outer products minus two cross products.
theorem lagrange_square_eq_expanded(a: Nat -> Real, b: Nat -> Real, i: Nat, j: Nat) {
    lagrange_square(a, b, i, j) = lagrange_expanded(a, b, i, j)
} by {
    let x = a(i) * b(j)
    let y = a(j) * b(i)
    lagrange_square(a, b, i, j) = (x - y) * (x - y)
    x - y = x + -y
    (x - y) * (x - y) = (x + -y) * (x + -y)
    real_square_add_expanded(x, -y)
    (x + -y) * (x + -y) = x * x + x * -y + (x * -y + -y * -y)
    mul_neg_right(x, y)
    x * -y = -(x * y)
    mul_neg_left(y, -y)
    -y * -y = -(y * -y)
    mul_neg_right(y, y)
    y * -y = -(y * y)
    -y * -y = -(-(y * y))
    -(-(y * y)) = y * y
    (x - y) * (x - y) = x * x + -(x * y) + (-(x * y) + y * y)
    product_pair_square_rearrange(a(i), b(j))
    x * x = (a(i) * a(i)) * (b(j) * b(j))
    product_pair_square_rearrange(a(j), b(i))
    y * y = (a(j) * a(j)) * (b(i) * b(i))
    y * y = (b(i) * b(i)) * (a(j) * a(j))
    product_cross_rearrange(a(i), b(j), a(j), b(i))
    x * y = (a(i) * b(i)) * (a(j) * b(j))
    pointwise_square(a, i) = a(i) * a(i)
    pointwise_square(b, j) = b(j) * b(j)
    pointwise_square(b, i) = b(i) * b(i)
    pointwise_square(a, j) = a(j) * a(j)
    pointwise_product(a, b, i) = a(i) * b(i)
    pointwise_product(a, b, j) = a(j) * b(j)
    square_outer(a, b, i, j) = pointwise_square(a, i) * pointwise_square(b, j)
    square_outer(b, a, i, j) = pointwise_square(b, i) * pointwise_square(a, j)
    dot_outer(a, b, i, j) = pointwise_product(a, b, i) * pointwise_product(a, b, j)
    lagrange_expanded(a, b, i, j) =
        square_outer(a, b, i, j) + -dot_outer(a, b, i, j) +
            (-dot_outer(a, b, i, j) + square_outer(b, a, i, j))
    lagrange_square(a, b, i, j) = lagrange_expanded(a, b, i, j)
}

/// Pointwise equality on a finite row gives equality of row sums.
theorem row_sum_pointwise_eq(
    f: (Nat, Nat) -> Real, g: (Nat, Nat) -> Real, i: Nat, n: Nat
) {
    (forall(j: Nat) {
        j < n implies f(i, j) = g(i, j)
    }) implies row_sum(f, n, i) = row_sum(g, n, i)
} by {
    if forall(j: Nat) {
        j < n implies f(i, j) = g(i, j)
    } {
        forall(j: Nat) {
            if j < n {
                f(i)(j) = f(i, j)
                g(i)(j) = g(i, j)
                f(i, j) = g(i, j)
                f(i)(j) = g(i)(j)
            }
        }
        partial_pointwise_eq(f(i), g(i), n)
        partial(f(i), n) = partial(g(i), n)
        row_sum(f, n, i) = partial(f(i), n)
        row_sum(g, n, i) = partial(g(i), n)
        row_sum(f, n, i) = row_sum(g, n, i)
    }
}

/// Equality of finite row sums gives equality of rectangular sums.
theorem rectangular_sum_row_eq(
    f: (Nat, Nat) -> Real, g: (Nat, Nat) -> Real, m: Nat, n: Nat
) {
    (forall(i: Nat) {
        i < m implies row_sum(f, n, i) = row_sum(g, n, i)
    }) implies rectangular_sum(f, m, n) = rectangular_sum(g, m, n)
} by {
    if forall(i: Nat) {
        i < m implies row_sum(f, n, i) = row_sum(g, n, i)
    } {
        partial_pointwise_eq(row_sum(f, n), row_sum(g, n), m)
        partial(row_sum(f, n), m) = partial(row_sum(g, n), m)
        rectangular_sum(f, m, n) = partial(row_sum(f, n), m)
        rectangular_sum(g, m, n) = partial(row_sum(g, n), m)
        rectangular_sum(f, m, n) = rectangular_sum(g, m, n)
    }
}

/// Pointwise equality on a finite rectangle gives equality of rectangular sums.
theorem rectangular_sum_pointwise_eq(
    f: (Nat, Nat) -> Real, g: (Nat, Nat) -> Real, m: Nat, n: Nat
) {
    (forall(i: Nat, j: Nat) {
        i < m and j < n implies f(i, j) = g(i, j)
    }) implies rectangular_sum(f, m, n) = rectangular_sum(g, m, n)
} by {
    if forall(i: Nat, j: Nat) {
        i < m and j < n implies f(i, j) = g(i, j)
    } {
        forall(i: Nat) {
            if i < m {
                forall(j: Nat) {
                    if j < n {
                        f(i, j) = g(i, j)
                    }
                }
                row_sum_pointwise_eq(f, g, i, n)
                row_sum(f, n, i) = row_sum(g, n, i)
            }
        }
        forall(i: Nat) {
            if i < m {
                row_sum(f, n, i) = row_sum(g, n, i)
            }
        }
        rectangular_sum_row_eq(f, g, m, n)
        rectangular_sum(f, m, n) = rectangular_sum(g, m, n)
    }
}

/// The rectangular sum of the square outer product factors into square sums.
theorem square_outer_sum_factors(a: Nat -> Real, b: Nat -> Real, n: Nat) {
    rectangular_sum(square_outer(a, b), n, n) =
        finite_square_sum(a, n) * finite_square_sum(b, n)
} by {
    forall(i: Nat, j: Nat) {
        square_outer(a, b, i, j) = pointwise_square(a, i) * pointwise_square(b, j)
        prod_fn(pointwise_square(a), pointwise_square(b), i, j) =
            pointwise_square(a, i) * pointwise_square(b, j)
        square_outer(a, b, i, j) = prod_fn(pointwise_square(a), pointwise_square(b), i, j)
    }
    square_outer(a, b) = prod_fn(pointwise_square(a), pointwise_square(b))
    rectangular_sum(square_outer(a, b), n, n) =
        rectangular_sum(prod_fn(pointwise_square(a), pointwise_square(b)), n, n)
    product_sum(pointwise_square(a), pointwise_square(b), n, n) =
        rectangular_sum(prod_fn(pointwise_square(a), pointwise_square(b)), n, n)
    product_sum_factors(pointwise_square(a), pointwise_square(b), n, n)
    product_sum(pointwise_square(a), pointwise_square(b), n, n) =
        partial(pointwise_square(a), n) * partial(pointwise_square(b), n)
    finite_square_sum(a, n) = partial(pointwise_square(a), n)
    finite_square_sum(b, n) = partial(pointwise_square(b), n)
}

/// The rectangular sum of the dot outer product factors into the square of a dot product.
theorem dot_outer_sum_factors(a: Nat -> Real, b: Nat -> Real, n: Nat) {
    rectangular_sum(dot_outer(a, b), n, n) = finite_dot(a, b, n) * finite_dot(a, b, n)
} by {
    forall(i: Nat, j: Nat) {
        dot_outer(a, b, i, j) = pointwise_product(a, b, i) * pointwise_product(a, b, j)
        prod_fn(pointwise_product(a, b), pointwise_product(a, b), i, j) =
            pointwise_product(a, b, i) * pointwise_product(a, b, j)
        dot_outer(a, b, i, j) = prod_fn(pointwise_product(a, b), pointwise_product(a, b), i, j)
    }
    dot_outer(a, b) = prod_fn(pointwise_product(a, b), pointwise_product(a, b))
    rectangular_sum(dot_outer(a, b), n, n) =
        rectangular_sum(prod_fn(pointwise_product(a, b), pointwise_product(a, b)), n, n)
    product_sum(pointwise_product(a, b), pointwise_product(a, b), n, n) =
        rectangular_sum(prod_fn(pointwise_product(a, b), pointwise_product(a, b)), n, n)
    product_sum_factors(pointwise_product(a, b), pointwise_product(a, b), n, n)
    product_sum(pointwise_product(a, b), pointwise_product(a, b), n, n) =
        partial(pointwise_product(a, b), n) * partial(pointwise_product(a, b), n)
    finite_dot(a, b, n) = partial(pointwise_product(a, b), n)
}

/// The rectangular sum of negative dot outer terms is the negative of the dot outer sum.
theorem neg_dot_outer_sum(a: Nat -> Real, b: Nat -> Real, n: Nat) {
    rectangular_sum(neg_dot_outer(a, b), n, n) = -rectangular_sum(dot_outer(a, b), n, n)
} by {
    forall(i: Nat, j: Nat) {
        neg_dot_outer(a, b, i, j) = -dot_outer(a, b, i, j)
        scalar_mul_fn_2(Real.0 - Real.1, dot_outer(a, b), i, j) =
            (Real.0 - Real.1) * dot_outer(a, b, i, j)
        Real.0 - Real.1 = -Real.1
        mul_neg_one_left(dot_outer(a, b, i, j))
        -Real.1 * dot_outer(a, b, i, j) = -dot_outer(a, b, i, j)
        scalar_mul_fn_2(Real.0 - Real.1, dot_outer(a, b), i, j) =
            -dot_outer(a, b, i, j)
        neg_dot_outer(a, b, i, j) = scalar_mul_fn_2(Real.0 - Real.1, dot_outer(a, b), i, j)
    }
    neg_dot_outer(a, b) = scalar_mul_fn_2(Real.0 - Real.1, dot_outer(a, b))
    rectangular_sum(neg_dot_outer(a, b), n, n) =
        rectangular_sum(scalar_mul_fn_2(Real.0 - Real.1, dot_outer(a, b)), n, n)
    rectangular_sum_scale(Real.0 - Real.1, dot_outer(a, b), n, n)
    rectangular_sum(scalar_mul_fn_2(Real.0 - Real.1, dot_outer(a, b)), n, n) =
        (Real.0 - Real.1) * rectangular_sum(dot_outer(a, b), n, n)
    Real.0 - Real.1 = -Real.1
    mul_neg_one_left(rectangular_sum(dot_outer(a, b), n, n))
    (Real.0 - Real.1) * rectangular_sum(dot_outer(a, b), n, n) =
        -rectangular_sum(dot_outer(a, b), n, n)
}

/// The rectangular sum of the first half of the expanded Lagrange term.
theorem lagrange_first_sum(a: Nat -> Real, b: Nat -> Real, n: Nat) {
    rectangular_sum(lagrange_first(a, b), n, n) =
        finite_square_sum(a, n) * finite_square_sum(b, n) +
            -(finite_dot(a, b, n) * finite_dot(a, b, n))
} by {
    forall(i: Nat, j: Nat) {
        lagrange_first(a, b, i, j) = square_outer(a, b, i, j) + neg_dot_outer(a, b, i, j)
        add_fn_2(square_outer(a, b), neg_dot_outer(a, b), i, j) =
            square_outer(a, b, i, j) + neg_dot_outer(a, b, i, j)
        lagrange_first(a, b, i, j) = add_fn_2(square_outer(a, b), neg_dot_outer(a, b), i, j)
    }
    lagrange_first(a, b) = add_fn_2(square_outer(a, b), neg_dot_outer(a, b))
    rectangular_sum(lagrange_first(a, b), n, n) =
        rectangular_sum(add_fn_2(square_outer(a, b), neg_dot_outer(a, b)), n, n)
    rectangular_sum_add(square_outer(a, b), neg_dot_outer(a, b), n, n)
    rectangular_sum(add_fn_2(square_outer(a, b), neg_dot_outer(a, b)), n, n) =
        rectangular_sum(square_outer(a, b), n, n) + rectangular_sum(neg_dot_outer(a, b), n, n)
    square_outer_sum_factors(a, b, n)
    dot_outer_sum_factors(a, b, n)
    neg_dot_outer_sum(a, b, n)
    rectangular_sum(square_outer(a, b), n, n) = finite_square_sum(a, n) * finite_square_sum(b, n)
    rectangular_sum(dot_outer(a, b), n, n) = finite_dot(a, b, n) * finite_dot(a, b, n)
    rectangular_sum(neg_dot_outer(a, b), n, n) = -(finite_dot(a, b, n) * finite_dot(a, b, n))
}

/// The rectangular sum of the second half of the expanded Lagrange term.
theorem lagrange_second_sum(a: Nat -> Real, b: Nat -> Real, n: Nat) {
    rectangular_sum(lagrange_second(a, b), n, n) =
        finite_square_sum(a, n) * finite_square_sum(b, n) +
            -(finite_dot(a, b, n) * finite_dot(a, b, n))
} by {
    forall(i: Nat, j: Nat) {
        lagrange_second(a, b, i, j) = neg_dot_outer(a, b, i, j) + square_outer(b, a, i, j)
        add_fn_2(neg_dot_outer(a, b), square_outer(b, a), i, j) =
            neg_dot_outer(a, b, i, j) + square_outer(b, a, i, j)
        lagrange_second(a, b, i, j) = add_fn_2(neg_dot_outer(a, b), square_outer(b, a), i, j)
    }
    lagrange_second(a, b) = add_fn_2(neg_dot_outer(a, b), square_outer(b, a))
    rectangular_sum(lagrange_second(a, b), n, n) =
        rectangular_sum(add_fn_2(neg_dot_outer(a, b), square_outer(b, a)), n, n)
    rectangular_sum_add(neg_dot_outer(a, b), square_outer(b, a), n, n)
    rectangular_sum(add_fn_2(neg_dot_outer(a, b), square_outer(b, a)), n, n) =
        rectangular_sum(neg_dot_outer(a, b), n, n) + rectangular_sum(square_outer(b, a), n, n)
    square_outer_sum_factors(b, a, n)
    dot_outer_sum_factors(a, b, n)
    neg_dot_outer_sum(a, b, n)
    rectangular_sum(square_outer(b, a), n, n) = finite_square_sum(b, n) * finite_square_sum(a, n)
    finite_square_sum(b, n) * finite_square_sum(a, n) = finite_square_sum(a, n) * finite_square_sum(b, n)
    rectangular_sum(dot_outer(a, b), n, n) = finite_dot(a, b, n) * finite_dot(a, b, n)
    rectangular_sum(neg_dot_outer(a, b), n, n) = -(finite_dot(a, b, n) * finite_dot(a, b, n))
    -(finite_dot(a, b, n) * finite_dot(a, b, n)) +
        finite_square_sum(a, n) * finite_square_sum(b, n) =
        finite_square_sum(a, n) * finite_square_sum(b, n) +
            -(finite_dot(a, b, n) * finite_dot(a, b, n))
}

/// The rectangular sum of the paired expanded Lagrange term is twice the Cauchy-Schwarz gap.
theorem lagrange_pair_sum(a: Nat -> Real, b: Nat -> Real, n: Nat) {
    rectangular_sum(lagrange_pair(a, b), n, n) =
        (finite_square_sum(a, n) * finite_square_sum(b, n) +
            -(finite_dot(a, b, n) * finite_dot(a, b, n))) +
        (finite_square_sum(a, n) * finite_square_sum(b, n) +
            -(finite_dot(a, b, n) * finite_dot(a, b, n)))
} by {
    forall(i: Nat, j: Nat) {
        lagrange_pair(a, b, i, j) = lagrange_first(a, b, i, j) + lagrange_second(a, b, i, j)
        add_fn_2(lagrange_first(a, b), lagrange_second(a, b), i, j) =
            lagrange_first(a, b, i, j) + lagrange_second(a, b, i, j)
        lagrange_pair(a, b, i, j) = add_fn_2(lagrange_first(a, b), lagrange_second(a, b), i, j)
    }
    lagrange_pair(a, b) = add_fn_2(lagrange_first(a, b), lagrange_second(a, b))
    rectangular_sum(lagrange_pair(a, b), n, n) =
        rectangular_sum(add_fn_2(lagrange_first(a, b), lagrange_second(a, b)), n, n)
    rectangular_sum_add(lagrange_first(a, b), lagrange_second(a, b), n, n)
    lagrange_first_sum(a, b, n)
    lagrange_second_sum(a, b, n)
}

/// The expanded Lagrange term agrees with its paired form.
theorem lagrange_expanded_eq_pair(a: Nat -> Real, b: Nat -> Real, i: Nat, j: Nat) {
    lagrange_expanded(a, b, i, j) = lagrange_pair(a, b, i, j)
} by {
    neg_dot_outer(a, b, i, j) = -dot_outer(a, b, i, j)
    lagrange_first(a, b, i, j) = square_outer(a, b, i, j) + neg_dot_outer(a, b, i, j)
    lagrange_second(a, b, i, j) = neg_dot_outer(a, b, i, j) + square_outer(b, a, i, j)
    lagrange_pair(a, b, i, j) = lagrange_first(a, b, i, j) + lagrange_second(a, b, i, j)
    lagrange_expanded(a, b, i, j) =
        square_outer(a, b, i, j) + -dot_outer(a, b, i, j) +
            (-dot_outer(a, b, i, j) + square_outer(b, a, i, j))
}

/// The sum of Lagrange square terms is twice the Cauchy-Schwarz gap.
theorem lagrange_square_sum_eq_gap_double(a: Nat -> Real, b: Nat -> Real, n: Nat) {
    rectangular_sum(lagrange_square(a, b), n, n) =
        (finite_square_sum(a, n) * finite_square_sum(b, n) +
            -(finite_dot(a, b, n) * finite_dot(a, b, n))) +
        (finite_square_sum(a, n) * finite_square_sum(b, n) +
            -(finite_dot(a, b, n) * finite_dot(a, b, n)))
} by {
    forall(i: Nat, j: Nat) {
        if i < n and j < n {
            lagrange_square_eq_expanded(a, b, i, j)
            lagrange_expanded_eq_pair(a, b, i, j)
            lagrange_square(a, b, i, j) = lagrange_pair(a, b, i, j)
        }
    }
    rectangular_sum_pointwise_eq(lagrange_square(a, b), lagrange_pair(a, b), n, n)
    rectangular_sum(lagrange_square(a, b), n, n) = rectangular_sum(lagrange_pair(a, b), n, n)
    lagrange_pair_sum(a, b, n)
}

/// If a doubled real number is nonnegative, then the original real number is nonnegative.
theorem double_nonnegative_imp_nonnegative(x: Real) {
    x + x >= Real.0 implies x >= Real.0
} by {
    if x + x >= Real.0 {
        one_half_positive
        Real.one_half >= Real.0
        mul_nonneg(Real.one_half, x + x)
        Real.one_half * (x + x) >= Real.0
        Real.one_half * (x + x) = Real.one_half * x + Real.one_half * x
        Real.one_half * x + Real.one_half * x = (Real.one_half + Real.one_half) * x
        one_half_plus_one_half
        Real.one_half + Real.one_half = Real.1
        (Real.one_half + Real.one_half) * x = Real.1 * x
        Real.1 * x = x
        x >= Real.0
    }
}

/// A nonnegative additive gap gives the corresponding order relation.
theorem nonnegative_gap_imp_lte(a: Real, b: Real) {
    a + -b >= Real.0 implies b <= a
} by {
    if a + -b >= Real.0 {
        Real.0 <= a + -b
        b + Real.0 <= b + (a + -b)
        a + -b = -b + a
        b + (a + -b) = b + (-b + a)
        b + (-b + a) = (b + -b) + a
        b + -b = Real.0
        (b + -b) + a = Real.0 + a
        Real.0 + a = a
        b + (a + -b) = a
        b <= a
    }
}

/// The finite real Cauchy-Schwarz inequality in squared form.
theorem finite_cauchy_schwarz(a: Nat -> Real, b: Nat -> Real, n: Nat) {
    finite_dot(a, b, n) * finite_dot(a, b, n) <= finite_square_sum(a, n) * finite_square_sum(b, n)
} by {
    let gap = finite_square_sum(a, n) * finite_square_sum(b, n) +
        -(finite_dot(a, b, n) * finite_dot(a, b, n))
    lagrange_square_sum_nonnegative(a, b, n)
    rectangular_sum(lagrange_square(a, b), n, n) >= Real.0
    lagrange_square_sum_eq_gap_double(a, b, n)
    rectangular_sum(lagrange_square(a, b), n, n) = gap + gap
    gap + gap >= Real.0
    double_nonnegative_imp_nonnegative(gap)
    gap >= Real.0
    nonnegative_gap_imp_lte(
        finite_square_sum(a, n) * finite_square_sum(b, n),
        finite_dot(a, b, n) * finite_dot(a, b, n)
    )
    finite_dot(a, b, n) * finite_dot(a, b, n) <= finite_square_sum(a, n) * finite_square_sum(b, n)
}
