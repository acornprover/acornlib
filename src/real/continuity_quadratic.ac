from real.continuity_affine import affine_real, continuous_affine_real,
    continuous_at_affine_real
from real.continuity_algebra import add_fns_continuous, add_fns_continuous_at
from real.continuity_const_mul import const_mul_left,
    continuous_at_const_mul_left, continuous_const_mul_left
from real.continuity_square import continuous_at_square_real,
    continuous_square_real, square_real
from real.continuity_base import Real, add_fns, continuous, continuous_at

/// The quadratic real function x -> a * x * x + b * x + c.
define quadratic_real(a: Real, b: Real, c: Real, x: Real) -> Real {
    a * x * x + b * x + c
}

/// The quadratic function agrees with the pointwise sum of a constant multiple of the square
/// function and the affine function x -> b * x + c.
theorem quadratic_real_eq_add_fns_const_mul_square_affine(a: Real, b: Real, c: Real) {
    quadratic_real(a, b, c) = add_fns(const_mul_left(a, square_real), affine_real(b, c))
} by {
    forall(x: Real) {
        square_real(x) = x * x
        const_mul_left(a, square_real, x) = a * square_real(x)
        const_mul_left(a, square_real, x) = a * (x * x)
        const_mul_left(a, square_real, x) = a * x * x
        affine_real(b, c, x) = b * x + c
        add_fns(const_mul_left(a, square_real), affine_real(b, c), x) = const_mul_left(a, square_real, x) + affine_real(b, c, x)
        add_fns(const_mul_left(a, square_real), affine_real(b, c), x) = a * x * x + (b * x + c)
        quadratic_real(a, b, c, x) = a * x * x + b * x + c
        quadratic_real(a, b, c, x) = add_fns(const_mul_left(a, square_real), affine_real(b, c), x)
    }
}

/// The quadratic function x -> a * x * x + b * x + c is continuous at each point.
theorem continuous_at_quadratic_real(a: Real, b: Real, c: Real, x: Real) {
    continuous_at(quadratic_real(a, b, c), x)
} by {
    continuous_at_square_real(x)
    continuous_at(square_real, x)
    continuous_at_const_mul_left(a, square_real, x)
    continuous_at(const_mul_left(a, square_real), x)
    continuous_at_affine_real(b, c, x)
    continuous_at(affine_real(b, c), x)
    add_fns_continuous_at(const_mul_left(a, square_real), affine_real(b, c), x)
    continuous_at(add_fns(const_mul_left(a, square_real), affine_real(b, c)), x)
    quadratic_real_eq_add_fns_const_mul_square_affine(a, b, c)
    continuous_at(quadratic_real(a, b, c), x)
}

/// The quadratic function x -> a * x * x + b * x + c is continuous.
theorem continuous_quadratic_real(a: Real, b: Real, c: Real) {
    continuous(quadratic_real(a, b, c))
} by {
    continuous_square_real
    continuous(square_real)
    continuous_const_mul_left(a, square_real)
    continuous(const_mul_left(a, square_real))
    continuous_affine_real(b, c)
    continuous(affine_real(b, c))
    add_fns_continuous(const_mul_left(a, square_real), affine_real(b, c))
    continuous(add_fns(const_mul_left(a, square_real), affine_real(b, c)))
    quadratic_real_eq_add_fns_const_mul_square_affine(a, b, c)
    continuous(quadratic_real(a, b, c))
}
