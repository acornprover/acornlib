/// Taylor's theorem with Lagrange remainder.
///
/// The mean value theorem says that a differentiable function's chord slope is
/// attained as a derivative value; Taylor's theorem is its n-th order
/// generalization: the value of an (n + 1)-times differentiable function at b
/// is its n-th order Taylor polynomial at a plus a remainder term
/// f^(n+1)(c) (b - a)^(n+1) / (n + 1)! for some interior point c.
///
/// This file states the general theorem and proves the first two instances:
/// the first order case is exactly the mean value theorem, and the second
/// order case (second derivative remainder) follows from Rolle's theorem
/// applied twice to a carefully chosen auxiliary function.  A general-n proof
/// would need iterated-derivative infrastructure (a recursive definition of
/// f^(k) and a factorial of reals), which is not yet present in the library.

from order import lt_trans, lt_imp_ne_symm
from order_set import closed_interval_set
from real.continuity_base import Real, continuous, continuous_at
from real.derivative_basic import has_derivative_at, has_derivative_at_unique,
    sub_ne_zero_of_ne, constant_has_derivative_at
from real.derivative_rules import derivative_pointwise_sub
from real.derivative_linear import derivative_pointwise_const_mul
from real.derivative_polynomial_affine import derivative_square_real_after_affine
from real.derivative_affine_named import affine_real_has_derivative_at, affine_real_eq_pointwise_affine
from real.derivative_continuity import div_mul_cancel_denominator
from real.calculus_api import is_derivative_fn, is_derivative_fn_at, is_derivative_fn_iff,
    is_derivative_fn_imp_continuous_at
from real.continuity_affine import affine_real, continuous_at_affine_real
from real.continuity_square import square_real, continuous_at_square_real
from real.continuity_pointwise import continuous_at_pointwise_neg, continuous_at_pointwise_add
from real.continuity_pointwise_mul import continuous_at_pointwise_mul
from real.continuity_composition import continuous_at_compose,
    constant_function_is_continuous_at
from real.real_base import add_comm, add_assoc, add_zero_right, add_neg_eq_zero, neg_zero
from real.real_ring import mul_zero_left, mul_zero_right, real_mul_comm, mul_assoc,
    mul_one_right, mul_one_left, mul_distrib_right, mul_distrib_left, mul_neg_right
from real.real_field import mul_inverse
from real.real_seq import sub_zero_imp_eq
from real.mean_value import continuous_on_closed, differentiable_on_open, is_derivative_on_open,
    is_derivative_on_open_imp_differentiable_on_open, rolle_theorem, mean_value_theorem,
    secant_slope
from data.basic.function_algebra import pointwise_add, pointwise_neg, pointwise_mul
from data.basic.functions import identity_fn, compose, function_extensionality,
    function_eq_transport_predicate_rev

numerals Real

/// The real number two, written as 1 + 1 (the digit Real.2 is not defined in
/// the library).
let taylor_two: Real = Real.1 + Real.1

/// The real number two is nonzero.
theorem taylor_two_not_zero {
    taylor_two != Real.0
} by {
    taylor_two = Real.1 + Real.1
    add_zero_right(Real.1)
    Real.1 + Real.0 = Real.1
    Real.0 != Real.1
    Real.1 != Real.0
    if taylor_two = Real.0 {
        taylor_two = Real.1 + Real.1
        Real.1 + Real.1 = Real.0
        false
    }
    taylor_two != Real.0
}

/// The first-order Taylor polynomial of f at a: f(a) + df(a) * (x - a).
define taylor2_lin(f: Real -> Real, df: Real -> Real, a: Real, x: Real) -> Real {
    f(a) + df(a) * (x - a)
}

/// The remainder of f after its first-order Taylor polynomial at a.
define taylor2_lin_remainder(f: Real -> Real, df: Real -> Real, a: Real, x: Real) -> Real {
    f(x) - taylor2_lin(f, df, a, x)
}

/// The squared distance from a: (x - a)^2.
define taylor2_sq(a: Real, x: Real) -> Real {
    square_real(x - a)
}

/// The second-order increment: f(b) - f(a) - df(a) * (b - a).
define taylor2_delta(f: Real -> Real, df: Real -> Real, a: Real, b: Real) -> Real {
    f(b) - f(a) - df(a) * (b - a)
}

/// The auxiliary function for the second-order Taylor theorem; it vanishes at
/// a and at b, so Rolle's theorem applies to it.
define taylor2_aux(f: Real -> Real, df: Real -> Real, a: Real, b: Real, x: Real) -> Real {
    square_real(b - a) * taylor2_lin_remainder(f, df, a, x) -
        taylor2_delta(f, df, a, b) * taylor2_sq(a, x)
}

/// The derivative of the auxiliary function of the second-order Taylor theorem.
define taylor2_aux_derivative(f: Real -> Real, df: Real -> Real, a: Real, b: Real, x: Real) -> Real {
    square_real(b - a) * (df(x) - df(a)) -
        taylor2_delta(f, df, a, b) * taylor_two * (x - a)
}

/// The second derivative of the auxiliary function of the second-order Taylor
/// theorem.
define taylor2_aux_second_derivative(f: Real -> Real, df: Real -> Real, ddf: Real -> Real, a: Real, b: Real, x: Real) -> Real {
    square_real(b - a) * ddf(x) -
        taylor2_delta(f, df, a, b) * taylor_two
}

/// Taylor's theorem of order one with Lagrange remainder: the mean value
/// theorem, restated in the Taylor form.
theorem taylor_order_one(f: Real -> Real, df: Real -> Real, a: Real, b: Real) {
    continuous(f) and a < b and is_derivative_fn(f, df)
    implies exists(c: Real) {
        a < c and c < b and f(b) = f(a) + df(c) * (b - a)
    }
} by {
    if continuous(f) and a < b and is_derivative_fn(f, df) {
        mean_value_theorem(f, df, a, b)
        let c: Real satisfy {
            a < c and c < b and has_derivative_at(f, c, secant_slope(f, a, b))
        }
        a < c and c < b
        has_derivative_at(f, c, secant_slope(f, a, b))
        is_derivative_fn_at(f, df, c)
        has_derivative_at(f, c, df(c))
        has_derivative_at_unique(f, c, secant_slope(f, a, b), df(c))
        secant_slope(f, a, b) = df(c)
        secant_slope(f, a, b) = (f(b) - f(a)) / (b - a)
        (f(b) - f(a)) / (b - a) = df(c)
        lt_imp_ne_symm(a, b)
        b != a
        sub_ne_zero_of_ne(b, a)
        b - a != Real.0
        div_mul_cancel_denominator(f(b) - f(a), b - a)
        ((f(b) - f(a)) / (b - a)) * (b - a) = f(b) - f(a)
        df(c) * (b - a) = ((f(b) - f(a)) / (b - a)) * (b - a)
        df(c) * (b - a) = f(b) - f(a)
        (f(b) - f(a)) + f(a) = (df(c) * (b - a)) + f(a)
        (f(b) - f(a)) + f(a) = f(b)
        (df(c) * (b - a)) + f(a) = f(a) + df(c) * (b - a)
        add_comm(df(c) * (b - a), f(a))
        df(c) * (b - a) + f(a) = f(a) + df(c) * (b - a)
        f(b) = f(a) + df(c) * (b - a)
        exists(c2: Real) {
            a < c2 and c2 < b and f(b) = f(a) + df(c2) * (b - a)
        }
    }
}

/// The auxiliary function of the second-order Taylor theorem agrees with a
/// pointwise combination of f, an affine function, and the square function.
theorem taylor2_aux_eq_pointwise(f: Real -> Real, df: Real -> Real, a: Real, b: Real) {
    taylor2_aux(f, df, a, b) = pointwise_add(
        pointwise_mul(constant[Real, Real](square_real(b - a)),
            pointwise_add(f, pointwise_neg(affine_real(df(a), f(a) - df(a) * a)))),
        pointwise_neg(pointwise_mul(constant[Real, Real](taylor2_delta(f, df, a, b)),
            compose(square_real, affine_real(Real.1, -a)))))
} by {
    forall(x: Real) {
        taylor2_aux(f, df, a, b, x) = square_real(b - a) * taylor2_lin_remainder(f, df, a, x) -
            taylor2_delta(f, df, a, b) * taylor2_sq(a, x)
        taylor2_lin_remainder(f, df, a, x) = f(x) - taylor2_lin(f, df, a, x)
        taylor2_lin(f, df, a, x) = f(a) + df(a) * (x - a)
        affine_real(df(a), f(a) - df(a) * a, x) = df(a) * x + (f(a) - df(a) * a)
        x - a = x + -a
        df(a) * (x - a) = df(a) * (x + -a)
        mul_distrib_right(df(a), x, -a)
        df(a) * (x + -a) = df(a) * x + df(a) * -a
        mul_neg_right(df(a), a)
        df(a) * -a = -(df(a) * a)
        df(a) * (x - a) = df(a) * x + -(df(a) * a)
        f(a) + df(a) * (x - a) = f(a) + (df(a) * x + -(df(a) * a))
        f(a) - df(a) * a = f(a) + -(df(a) * a)
        df(a) * x + (f(a) - df(a) * a) = df(a) * x + (f(a) + -(df(a) * a))
        add_comm(f(a), df(a) * x)
        f(a) + df(a) * x = df(a) * x + f(a)
        add_assoc(f(a), df(a) * x, -(df(a) * a))
        (f(a) + df(a) * x) + -(df(a) * a) = f(a) + (df(a) * x + -(df(a) * a))
        add_assoc(df(a) * x, f(a), -(df(a) * a))
        (df(a) * x + f(a)) + -(df(a) * a) = df(a) * x + (f(a) + -(df(a) * a))
        f(a) + (df(a) * x + -(df(a) * a)) =
            df(a) * x + (f(a) + -(df(a) * a))
        f(a) + df(a) * (x - a) = df(a) * x + (f(a) - df(a) * a)
        taylor2_lin(f, df, a, x) = affine_real(df(a), f(a) - df(a) * a, x)
        taylor2_lin_remainder(f, df, a, x) = f(x) - affine_real(df(a), f(a) - df(a) * a, x)
        f(x) - affine_real(df(a), f(a) - df(a) * a, x) =
            f(x) + -(affine_real(df(a), f(a) - df(a) * a, x))
        pointwise_neg(affine_real(df(a), f(a) - df(a) * a), x) =
            -(affine_real(df(a), f(a) - df(a) * a, x))
        pointwise_add(f, pointwise_neg(affine_real(df(a), f(a) - df(a) * a)), x) =
            f(x) + pointwise_neg(affine_real(df(a), f(a) - df(a) * a), x)
        pointwise_add(f, pointwise_neg(affine_real(df(a), f(a) - df(a) * a)), x) =
            f(x) + -(affine_real(df(a), f(a) - df(a) * a, x))
        taylor2_lin_remainder(f, df, a, x) =
            pointwise_add(f, pointwise_neg(affine_real(df(a), f(a) - df(a) * a)), x)
        taylor2_sq(a, x) = square_real(x - a)
        affine_real(Real.1, -a, x) = Real.1 * x + -a
        mul_one_left(x)
        Real.1 * x = x
        x + -a = x - a
        Real.1 * x + -a = x - a
        square_real(affine_real(Real.1, -a, x)) = square_real(x - a)
        compose(square_real, affine_real(Real.1, -a), x) =
            square_real(affine_real(Real.1, -a, x))
        compose(square_real, affine_real(Real.1, -a), x) = square_real(x - a)
        taylor2_sq(a, x) = compose(square_real, affine_real(Real.1, -a), x)
        pointwise_mul(constant[Real, Real](square_real(b - a)),
            pointwise_add(f, pointwise_neg(affine_real(df(a), f(a) - df(a) * a))), x) =
            constant[Real, Real](square_real(b - a), x) *
                pointwise_add(f, pointwise_neg(affine_real(df(a), f(a) - df(a) * a)), x)
        constant[Real, Real](square_real(b - a), x) = square_real(b - a)
        pointwise_mul(constant[Real, Real](square_real(b - a)),
            pointwise_add(f, pointwise_neg(affine_real(df(a), f(a) - df(a) * a))), x) =
            square_real(b - a) * taylor2_lin_remainder(f, df, a, x)
        pointwise_mul(constant[Real, Real](taylor2_delta(f, df, a, b)),
            compose(square_real, affine_real(Real.1, -a)), x) =
            constant[Real, Real](taylor2_delta(f, df, a, b), x) *
                compose(square_real, affine_real(Real.1, -a), x)
        constant[Real, Real](taylor2_delta(f, df, a, b), x) = taylor2_delta(f, df, a, b)
        pointwise_mul(constant[Real, Real](taylor2_delta(f, df, a, b)),
            compose(square_real, affine_real(Real.1, -a)), x) =
            taylor2_delta(f, df, a, b) * taylor2_sq(a, x)
        pointwise_neg(pointwise_mul(constant[Real, Real](taylor2_delta(f, df, a, b)),
            compose(square_real, affine_real(Real.1, -a))), x) =
            -(pointwise_mul(constant[Real, Real](taylor2_delta(f, df, a, b)),
                compose(square_real, affine_real(Real.1, -a)), x))
        pointwise_neg(pointwise_mul(constant[Real, Real](taylor2_delta(f, df, a, b)),
            compose(square_real, affine_real(Real.1, -a))), x) =
            -(taylor2_delta(f, df, a, b) * taylor2_sq(a, x))
        pointwise_add(pointwise_mul(constant[Real, Real](square_real(b - a)),
                pointwise_add(f, pointwise_neg(affine_real(df(a), f(a) - df(a) * a)))),
            pointwise_neg(pointwise_mul(constant[Real, Real](taylor2_delta(f, df, a, b)),
                compose(square_real, affine_real(Real.1, -a)))), x) =
            pointwise_mul(constant[Real, Real](square_real(b - a)),
                pointwise_add(f, pointwise_neg(affine_real(df(a), f(a) - df(a) * a))), x) +
            pointwise_neg(pointwise_mul(constant[Real, Real](taylor2_delta(f, df, a, b)),
                compose(square_real, affine_real(Real.1, -a))), x)
        pointwise_add(pointwise_mul(constant[Real, Real](square_real(b - a)),
                pointwise_add(f, pointwise_neg(affine_real(df(a), f(a) - df(a) * a)))),
            pointwise_neg(pointwise_mul(constant[Real, Real](taylor2_delta(f, df, a, b)),
                compose(square_real, affine_real(Real.1, -a)))), x) =
            square_real(b - a) * taylor2_lin_remainder(f, df, a, x) -
                taylor2_delta(f, df, a, b) * taylor2_sq(a, x)
        taylor2_aux(f, df, a, b, x) = pointwise_add(pointwise_mul(constant[Real, Real](square_real(b - a)),
                pointwise_add(f, pointwise_neg(affine_real(df(a), f(a) - df(a) * a)))),
            pointwise_neg(pointwise_mul(constant[Real, Real](taylor2_delta(f, df, a, b)),
                compose(square_real, affine_real(Real.1, -a)))), x)
    }
    function_extensionality(taylor2_aux(f, df, a, b),
        pointwise_add(pointwise_mul(constant[Real, Real](square_real(b - a)),
                pointwise_add(f, pointwise_neg(affine_real(df(a), f(a) - df(a) * a)))),
            pointwise_neg(pointwise_mul(constant[Real, Real](taylor2_delta(f, df, a, b)),
                compose(square_real, affine_real(Real.1, -a))))))
    taylor2_aux(f, df, a, b) = pointwise_add(pointwise_mul(constant[Real, Real](square_real(b - a)),
            pointwise_add(f, pointwise_neg(affine_real(df(a), f(a) - df(a) * a)))),
        pointwise_neg(pointwise_mul(constant[Real, Real](taylor2_delta(f, df, a, b)),
            compose(square_real, affine_real(Real.1, -a)))))
}

/// The derivative of the auxiliary function agrees with a pointwise
/// combination of df, a constant, and an affine function.
theorem taylor2_aux_derivative_eq_pointwise(f: Real -> Real, df: Real -> Real, a: Real, b: Real) {
    taylor2_aux_derivative(f, df, a, b) = pointwise_add(
        pointwise_mul(constant[Real, Real](square_real(b - a)),
            pointwise_add(df, pointwise_neg(constant[Real, Real](df(a))))),
        pointwise_neg(pointwise_mul(constant[Real, Real](taylor2_delta(f, df, a, b) * taylor_two),
            affine_real(Real.1, -a))))
} by {
    forall(x: Real) {
        taylor2_aux_derivative(f, df, a, b, x) = square_real(b - a) * (df(x) - df(a)) -
            taylor2_delta(f, df, a, b) * taylor_two * (x - a)
        pointwise_neg(constant[Real, Real](df(a)), x) = -(constant[Real, Real](df(a), x))
        constant[Real, Real](df(a), x) = df(a)
        pointwise_neg(constant[Real, Real](df(a)), x) = -df(a)
        pointwise_add(df, pointwise_neg(constant[Real, Real](df(a))), x) =
            df(x) + pointwise_neg(constant[Real, Real](df(a)), x)
        pointwise_add(df, pointwise_neg(constant[Real, Real](df(a))), x) = df(x) - df(a)
        pointwise_mul(constant[Real, Real](square_real(b - a)),
            pointwise_add(df, pointwise_neg(constant[Real, Real](df(a)))), x) =
            constant[Real, Real](square_real(b - a), x) *
                pointwise_add(df, pointwise_neg(constant[Real, Real](df(a))), x)
        constant[Real, Real](square_real(b - a), x) = square_real(b - a)
        pointwise_mul(constant[Real, Real](square_real(b - a)),
            pointwise_add(df, pointwise_neg(constant[Real, Real](df(a)))), x) =
            square_real(b - a) * (df(x) - df(a))
        affine_real(Real.1, -a, x) = Real.1 * x + -a
        mul_one_left(x)
        Real.1 * x = x
        x + -a = x - a
        Real.1 * x + -a = x - a
        pointwise_mul(constant[Real, Real](taylor2_delta(f, df, a, b) * taylor_two),
            affine_real(Real.1, -a), x) =
            constant[Real, Real](taylor2_delta(f, df, a, b) * taylor_two, x) *
                affine_real(Real.1, -a, x)
        constant[Real, Real](taylor2_delta(f, df, a, b) * taylor_two, x) =
            taylor2_delta(f, df, a, b) * taylor_two
        pointwise_mul(constant[Real, Real](taylor2_delta(f, df, a, b) * taylor_two),
            affine_real(Real.1, -a), x) = taylor2_delta(f, df, a, b) * taylor_two * (x - a)
        pointwise_neg(pointwise_mul(constant[Real, Real](taylor2_delta(f, df, a, b) * taylor_two),
            affine_real(Real.1, -a)), x) =
            -(pointwise_mul(constant[Real, Real](taylor2_delta(f, df, a, b) * taylor_two),
                affine_real(Real.1, -a), x))
        pointwise_neg(pointwise_mul(constant[Real, Real](taylor2_delta(f, df, a, b) * taylor_two),
            affine_real(Real.1, -a)), x) =
            -(taylor2_delta(f, df, a, b) * taylor_two * (x - a))
        pointwise_add(pointwise_mul(constant[Real, Real](square_real(b - a)),
                pointwise_add(df, pointwise_neg(constant[Real, Real](df(a))))),
            pointwise_neg(pointwise_mul(constant[Real, Real](taylor2_delta(f, df, a, b) * taylor_two),
                affine_real(Real.1, -a))), x) =
            square_real(b - a) * (df(x) - df(a)) -
                taylor2_delta(f, df, a, b) * taylor_two * (x - a)
        taylor2_aux_derivative(f, df, a, b, x) = pointwise_add(
            pointwise_mul(constant[Real, Real](square_real(b - a)),
                pointwise_add(df, pointwise_neg(constant[Real, Real](df(a))))),
            pointwise_neg(pointwise_mul(constant[Real, Real](taylor2_delta(f, df, a, b) * taylor_two),
                affine_real(Real.1, -a))), x)
    }
    function_extensionality(taylor2_aux_derivative(f, df, a, b),
        pointwise_add(pointwise_mul(constant[Real, Real](square_real(b - a)),
                pointwise_add(df, pointwise_neg(constant[Real, Real](df(a))))),
            pointwise_neg(pointwise_mul(constant[Real, Real](taylor2_delta(f, df, a, b) * taylor_two),
                affine_real(Real.1, -a)))))
    taylor2_aux_derivative(f, df, a, b) = pointwise_add(
        pointwise_mul(constant[Real, Real](square_real(b - a)),
            pointwise_add(df, pointwise_neg(constant[Real, Real](df(a))))),
        pointwise_neg(pointwise_mul(constant[Real, Real](taylor2_delta(f, df, a, b) * taylor_two),
            affine_real(Real.1, -a))))
}

/// The auxiliary function of the second-order Taylor theorem is continuous on
/// every closed interval when f is differentiable everywhere.
theorem taylor2_aux_continuous_on_closed(f: Real -> Real, df: Real -> Real, a: Real, b: Real, u: Real, v: Real) {
    is_derivative_fn(f, df)
    implies continuous_on_closed(taylor2_aux(f, df, a, b), u, v)
} by {
    if is_derivative_fn(f, df) {
        forall(x: Real) {
            if closed_interval_set(u, v).contains(x) {
                is_derivative_fn_imp_continuous_at(f, df, x)
                continuous_at(f, x)
                continuous_at_affine_real(df(a), f(a) - df(a) * a, x)
                continuous_at(affine_real(df(a), f(a) - df(a) * a), x)
                continuous_at_pointwise_neg(affine_real(df(a), f(a) - df(a) * a), x)
                continuous_at(pointwise_neg(affine_real(df(a), f(a) - df(a) * a)), x)
                continuous_at_pointwise_add(f,
                    pointwise_neg(affine_real(df(a), f(a) - df(a) * a)), x)
                continuous_at(pointwise_add(f,
                    pointwise_neg(affine_real(df(a), f(a) - df(a) * a))), x)
                constant_function_is_continuous_at(square_real(b - a), x)
                continuous_at(constant[Real, Real](square_real(b - a)), x)
                continuous_at_pointwise_mul(constant[Real, Real](square_real(b - a)),
                    pointwise_add(f, pointwise_neg(affine_real(df(a), f(a) - df(a) * a))), x)
                continuous_at(pointwise_mul(constant[Real, Real](square_real(b - a)),
                    pointwise_add(f, pointwise_neg(affine_real(df(a), f(a) - df(a) * a)))), x)
                continuous_at_affine_real(Real.1, -a, x)
                continuous_at(affine_real(Real.1, -a), x)
                continuous_at_square_real(affine_real(Real.1, -a, x))
                continuous_at(square_real, affine_real(Real.1, -a, x))
                continuous_at_compose(square_real, affine_real(Real.1, -a), x)
                continuous_at(compose(square_real, affine_real(Real.1, -a)), x)
                constant_function_is_continuous_at(taylor2_delta(f, df, a, b), x)
                continuous_at(constant[Real, Real](taylor2_delta(f, df, a, b)), x)
                continuous_at_pointwise_mul(constant[Real, Real](taylor2_delta(f, df, a, b)),
                    compose(square_real, affine_real(Real.1, -a)), x)
                continuous_at(pointwise_mul(constant[Real, Real](taylor2_delta(f, df, a, b)),
                    compose(square_real, affine_real(Real.1, -a))), x)
                continuous_at_pointwise_neg(pointwise_mul(constant[Real, Real](taylor2_delta(f, df, a, b)),
                    compose(square_real, affine_real(Real.1, -a))), x)
                continuous_at(pointwise_neg(pointwise_mul(constant[Real, Real](taylor2_delta(f, df, a, b)),
                    compose(square_real, affine_real(Real.1, -a)))), x)
                continuous_at_pointwise_add(pointwise_mul(constant[Real, Real](square_real(b - a)),
                        pointwise_add(f, pointwise_neg(affine_real(df(a), f(a) - df(a) * a)))),
                    pointwise_neg(pointwise_mul(constant[Real, Real](taylor2_delta(f, df, a, b)),
                        compose(square_real, affine_real(Real.1, -a)))), x)
                continuous_at(pointwise_add(pointwise_mul(constant[Real, Real](square_real(b - a)),
                        pointwise_add(f, pointwise_neg(affine_real(df(a), f(a) - df(a) * a)))),
                    pointwise_neg(pointwise_mul(constant[Real, Real](taylor2_delta(f, df, a, b)),
                        compose(square_real, affine_real(Real.1, -a))))), x)
                taylor2_aux_eq_pointwise(f, df, a, b)
                taylor2_aux(f, df, a, b) = pointwise_add(pointwise_mul(constant[Real, Real](square_real(b - a)),
                        pointwise_add(f, pointwise_neg(affine_real(df(a), f(a) - df(a) * a)))),
                    pointwise_neg(pointwise_mul(constant[Real, Real](taylor2_delta(f, df, a, b)),
                        compose(square_real, affine_real(Real.1, -a)))))
                function_eq_transport_predicate_rev(
                    function(h: Real -> Real) { continuous_at(h, x) },
                    taylor2_aux(f, df, a, b),
                    pointwise_add(pointwise_mul(constant[Real, Real](square_real(b - a)),
                            pointwise_add(f, pointwise_neg(affine_real(df(a), f(a) - df(a) * a)))),
                        pointwise_neg(pointwise_mul(constant[Real, Real](taylor2_delta(f, df, a, b)),
                            compose(square_real, affine_real(Real.1, -a))))))
                continuous_at(taylor2_aux(f, df, a, b), x)
            }
        }
        continuous_on_closed(taylor2_aux(f, df, a, b), u, v) = forall(x: Real) {
            closed_interval_set(u, v).contains(x) implies continuous_at(taylor2_aux(f, df, a, b), x)
        }
        continuous_on_closed(taylor2_aux(f, df, a, b), u, v)
    }
}

/// A global derivative function is a pointwise derivative on every open interval.
theorem is_derivative_fn_imp_is_derivative_on_open(
    f: Real -> Real, df: Real -> Real, a: Real, b: Real
) {
    is_derivative_fn(f, df) implies is_derivative_on_open(f, df, a, b)
} by {
    if is_derivative_fn(f, df) {
        is_derivative_fn_iff(f, df)
        forall(x: Real) {
            has_derivative_at(f, x, df(x))
        }
        forall(x: Real) {
            if a < x and x < b {
                forall(y: Real) {
                    has_derivative_at(f, y, df(y))
                }
                has_derivative_at(f, x, df(x))
            }
        }
        is_derivative_on_open(f, df, a, b) = forall(x: Real) {
            a < x and x < b implies has_derivative_at(f, x, df(x))
        }
        is_derivative_on_open(f, df, a, b)
    }
}

/// The square of the shift x ↦ (x - a)^2 has derivative 2 (x - a) at every point.
theorem taylor2_sq_has_derivative_at(a: Real, x0: Real) {
    has_derivative_at(compose(square_real, affine_real(Real.1, -a)), x0, taylor_two * (x0 - a))
} by {
    let affine = pointwise_add(pointwise_mul(constant[Real, Real](Real.1), identity_fn[Real]), constant[Real, Real](-a))
    derivative_square_real_after_affine(Real.1, -a, x0)
    has_derivative_at(compose(square_real, affine), x0,
        (affine(x0) * Real.1 + affine(x0) * Real.1) * Real.1)
    affine_real_eq_pointwise_affine(Real.1, -a)
    affine_real(Real.1, -a) = affine
    function_eq_transport_predicate_rev(
        function(h: Real -> Real) {
            has_derivative_at(compose(square_real, h), x0,
                (affine(x0) * Real.1 + affine(x0) * Real.1) * Real.1)
        },
        affine_real(Real.1, -a), affine)
    has_derivative_at(compose(square_real, affine_real(Real.1, -a)), x0,
        (affine(x0) * Real.1 + affine(x0) * Real.1) * Real.1)
    affine(x0) = Real.1 * x0 + -a
    mul_one_left(x0)
    Real.1 * x0 = x0
    x0 + -a = x0 - a
    affine(x0) = x0 - a
    mul_one_right(affine(x0))
    affine(x0) * Real.1 = affine(x0)
    affine(x0) * Real.1 + affine(x0) * Real.1 = affine(x0) + affine(x0)
    mul_one_left(affine(x0))
    Real.1 * affine(x0) = affine(x0)
    mul_distrib_left(Real.1, Real.1, affine(x0))
    (Real.1 + Real.1) * affine(x0) = Real.1 * affine(x0) + Real.1 * affine(x0)
    (Real.1 + Real.1) * affine(x0) = affine(x0) + affine(x0)
    taylor_two = Real.1 + Real.1
    taylor_two * affine(x0) = (Real.1 + Real.1) * affine(x0)
    taylor_two * affine(x0) = affine(x0) + affine(x0)
    affine(x0) * Real.1 + affine(x0) * Real.1 = taylor_two * affine(x0)
    (affine(x0) * Real.1 + affine(x0) * Real.1) * Real.1 =
        taylor_two * affine(x0) * Real.1
    mul_one_right(taylor_two * affine(x0))
    (taylor_two * affine(x0)) * Real.1 = taylor_two * affine(x0)
    (affine(x0) * Real.1 + affine(x0) * Real.1) * Real.1 =
        taylor_two * affine(x0)
    taylor_two * affine(x0) = taylor_two * (x0 - a)
    (affine(x0) * Real.1 + affine(x0) * Real.1) * Real.1 =
        taylor_two * (x0 - a)
    has_derivative_at(compose(square_real, affine_real(Real.1, -a)), x0,
        taylor_two * (x0 - a))
}

/// The derivative of the auxiliary function of the second-order Taylor theorem
/// is continuous on every closed interval when df is differentiable everywhere.
theorem taylor2_aux_derivative_continuous_on_closed(f: Real -> Real, df: Real -> Real, ddf: Real -> Real, a: Real, b: Real, u: Real, v: Real) {
    is_derivative_fn(df, ddf)
    implies continuous_on_closed(taylor2_aux_derivative(f, df, a, b), u, v)
} by {
    if is_derivative_fn(df, ddf) {
        forall(x: Real) {
            if closed_interval_set(u, v).contains(x) {
                is_derivative_fn_imp_continuous_at(df, ddf, x)
                continuous_at(df, x)
                constant_function_is_continuous_at(df(a), x)
                continuous_at(constant[Real, Real](df(a)), x)
                continuous_at_pointwise_neg(constant[Real, Real](df(a)), x)
                continuous_at(pointwise_neg(constant[Real, Real](df(a))), x)
                continuous_at_pointwise_add(df,
                    pointwise_neg(constant[Real, Real](df(a))), x)
                continuous_at(pointwise_add(df,
                    pointwise_neg(constant[Real, Real](df(a)))), x)
                constant_function_is_continuous_at(square_real(b - a), x)
                continuous_at(constant[Real, Real](square_real(b - a)), x)
                continuous_at_pointwise_mul(constant[Real, Real](square_real(b - a)),
                    pointwise_add(df, pointwise_neg(constant[Real, Real](df(a)))), x)
                continuous_at(pointwise_mul(constant[Real, Real](square_real(b - a)),
                    pointwise_add(df, pointwise_neg(constant[Real, Real](df(a))))), x)
                continuous_at_affine_real(Real.1, -a, x)
                continuous_at(affine_real(Real.1, -a), x)
                constant_function_is_continuous_at(taylor2_delta(f, df, a, b) * taylor_two, x)
                continuous_at(constant[Real, Real](taylor2_delta(f, df, a, b) * taylor_two), x)
                continuous_at_pointwise_mul(constant[Real, Real](taylor2_delta(f, df, a, b) * taylor_two),
                    affine_real(Real.1, -a), x)
                continuous_at(pointwise_mul(constant[Real, Real](taylor2_delta(f, df, a, b) * taylor_two),
                    affine_real(Real.1, -a)), x)
                continuous_at_pointwise_neg(pointwise_mul(constant[Real, Real](taylor2_delta(f, df, a, b) * taylor_two),
                    affine_real(Real.1, -a)), x)
                continuous_at(pointwise_neg(pointwise_mul(constant[Real, Real](taylor2_delta(f, df, a, b) * taylor_two),
                    affine_real(Real.1, -a))), x)
                continuous_at_pointwise_add(pointwise_mul(constant[Real, Real](square_real(b - a)),
                        pointwise_add(df, pointwise_neg(constant[Real, Real](df(a))))),
                    pointwise_neg(pointwise_mul(constant[Real, Real](taylor2_delta(f, df, a, b) * taylor_two),
                        affine_real(Real.1, -a))), x)
                continuous_at(pointwise_add(pointwise_mul(constant[Real, Real](square_real(b - a)),
                        pointwise_add(df, pointwise_neg(constant[Real, Real](df(a))))),
                    pointwise_neg(pointwise_mul(constant[Real, Real](taylor2_delta(f, df, a, b) * taylor_two),
                        affine_real(Real.1, -a)))), x)
                taylor2_aux_derivative_eq_pointwise(f, df, a, b)
                taylor2_aux_derivative(f, df, a, b) = pointwise_add(
                    pointwise_mul(constant[Real, Real](square_real(b - a)),
                        pointwise_add(df, pointwise_neg(constant[Real, Real](df(a))))),
                    pointwise_neg(pointwise_mul(constant[Real, Real](taylor2_delta(f, df, a, b) * taylor_two),
                        affine_real(Real.1, -a))))
                function_eq_transport_predicate_rev(
                    function(h: Real -> Real) { continuous_at(h, x) },
                    taylor2_aux_derivative(f, df, a, b),
                    pointwise_add(pointwise_mul(constant[Real, Real](square_real(b - a)),
                            pointwise_add(df, pointwise_neg(constant[Real, Real](df(a))))),
                        pointwise_neg(pointwise_mul(constant[Real, Real](taylor2_delta(f, df, a, b) * taylor_two),
                            affine_real(Real.1, -a)))))
                continuous_at(taylor2_aux_derivative(f, df, a, b), x)
            }
        }
        continuous_on_closed(taylor2_aux_derivative(f, df, a, b), u, v) = forall(x: Real) {
            closed_interval_set(u, v).contains(x) implies
                continuous_at(taylor2_aux_derivative(f, df, a, b), x)
        }
        continuous_on_closed(taylor2_aux_derivative(f, df, a, b), u, v)
    }
}

/// The auxiliary function of the second-order Taylor theorem has the expected
/// global derivative function.
theorem taylor2_aux_is_derivative_fn(f: Real -> Real, df: Real -> Real, a: Real, b: Real) {
    is_derivative_fn(f, df)
    implies is_derivative_fn(taylor2_aux(f, df, a, b), taylor2_aux_derivative(f, df, a, b))
} by {
    if is_derivative_fn(f, df) {
        forall(x: Real) {
            is_derivative_fn_at(f, df, x)
            has_derivative_at(f, x, df(x))
            affine_real_has_derivative_at(df(a), f(a) - df(a) * a, x)
            has_derivative_at(affine_real(df(a), f(a) - df(a) * a), x, df(a))
            derivative_pointwise_sub(f, affine_real(df(a), f(a) - df(a) * a),
                x, df(x), df(a))
            has_derivative_at(pointwise_add(f,
                pointwise_neg(affine_real(df(a), f(a) - df(a) * a))), x, df(x) + -df(a))
            df(x) + -df(a) = df(x) - df(a)
            has_derivative_at(pointwise_add(f,
                pointwise_neg(affine_real(df(a), f(a) - df(a) * a))), x, df(x) - df(a))
            derivative_pointwise_const_mul(square_real(b - a),
                pointwise_add(f, pointwise_neg(affine_real(df(a), f(a) - df(a) * a))),
                x, df(x) - df(a))
            has_derivative_at(pointwise_mul(constant[Real, Real](square_real(b - a)),
                pointwise_add(f, pointwise_neg(affine_real(df(a), f(a) - df(a) * a)))),
                x, square_real(b - a) * (df(x) - df(a)))
            taylor2_sq_has_derivative_at(a, x)
            has_derivative_at(compose(square_real, affine_real(Real.1, -a)), x,
                taylor_two * (x - a))
            derivative_pointwise_const_mul(taylor2_delta(f, df, a, b),
                compose(square_real, affine_real(Real.1, -a)), x, taylor_two * (x - a))
            has_derivative_at(pointwise_mul(constant[Real, Real](taylor2_delta(f, df, a, b)),
                compose(square_real, affine_real(Real.1, -a))), x,
                taylor2_delta(f, df, a, b) * (taylor_two * (x - a)))
            mul_assoc(taylor2_delta(f, df, a, b), taylor_two, x - a)
            taylor2_delta(f, df, a, b) * (taylor_two * (x - a)) =
                taylor2_delta(f, df, a, b) * taylor_two * (x - a)
            has_derivative_at(pointwise_mul(constant[Real, Real](taylor2_delta(f, df, a, b)),
                compose(square_real, affine_real(Real.1, -a))), x,
                taylor2_delta(f, df, a, b) * taylor_two * (x - a))
            derivative_pointwise_sub(
                pointwise_mul(constant[Real, Real](square_real(b - a)),
                    pointwise_add(f, pointwise_neg(affine_real(df(a), f(a) - df(a) * a)))),
                pointwise_mul(constant[Real, Real](taylor2_delta(f, df, a, b)),
                    compose(square_real, affine_real(Real.1, -a))),
                x, square_real(b - a) * (df(x) - df(a)),
                taylor2_delta(f, df, a, b) * taylor_two * (x - a))
            has_derivative_at(pointwise_add(
                pointwise_mul(constant[Real, Real](square_real(b - a)),
                    pointwise_add(f, pointwise_neg(affine_real(df(a), f(a) - df(a) * a)))),
                pointwise_neg(pointwise_mul(constant[Real, Real](taylor2_delta(f, df, a, b)),
                    compose(square_real, affine_real(Real.1, -a))))),
                x, square_real(b - a) * (df(x) - df(a)) +
                    -(taylor2_delta(f, df, a, b) * taylor_two * (x - a)))
            square_real(b - a) * (df(x) - df(a)) +
                -(taylor2_delta(f, df, a, b) * taylor_two * (x - a)) =
                square_real(b - a) * (df(x) - df(a)) -
                    taylor2_delta(f, df, a, b) * taylor_two * (x - a)
            has_derivative_at(pointwise_add(
                pointwise_mul(constant[Real, Real](square_real(b - a)),
                    pointwise_add(f, pointwise_neg(affine_real(df(a), f(a) - df(a) * a)))),
                pointwise_neg(pointwise_mul(constant[Real, Real](taylor2_delta(f, df, a, b)),
                    compose(square_real, affine_real(Real.1, -a))))),
                x, square_real(b - a) * (df(x) - df(a)) -
                    taylor2_delta(f, df, a, b) * taylor_two * (x - a))
            taylor2_aux_derivative(f, df, a, b, x) =
                square_real(b - a) * (df(x) - df(a)) -
                    taylor2_delta(f, df, a, b) * taylor_two * (x - a)
            taylor2_aux_eq_pointwise(f, df, a, b)
            taylor2_aux(f, df, a, b) = pointwise_add(
                pointwise_mul(constant[Real, Real](square_real(b - a)),
                    pointwise_add(f, pointwise_neg(affine_real(df(a), f(a) - df(a) * a)))),
                pointwise_neg(pointwise_mul(constant[Real, Real](taylor2_delta(f, df, a, b)),
                    compose(square_real, affine_real(Real.1, -a)))))
            function_eq_transport_predicate_rev(
                function(h: Real -> Real) {
                    has_derivative_at(h, x, taylor2_aux_derivative(f, df, a, b, x))
                },
                taylor2_aux(f, df, a, b),
                pointwise_add(pointwise_mul(constant[Real, Real](square_real(b - a)),
                        pointwise_add(f, pointwise_neg(affine_real(df(a), f(a) - df(a) * a)))),
                    pointwise_neg(pointwise_mul(constant[Real, Real](taylor2_delta(f, df, a, b)),
                        compose(square_real, affine_real(Real.1, -a))))))
            has_derivative_at(taylor2_aux(f, df, a, b), x,
                taylor2_aux_derivative(f, df, a, b, x))
        }
        is_derivative_fn_iff(taylor2_aux(f, df, a, b), taylor2_aux_derivative(f, df, a, b))
        is_derivative_fn(taylor2_aux(f, df, a, b), taylor2_aux_derivative(f, df, a, b))
    }
}

/// The derivative of the auxiliary function of the second-order Taylor theorem
/// has the expected global derivative function.
theorem taylor2_aux_derivative_is_derivative_fn(f: Real -> Real, df: Real -> Real, ddf: Real -> Real, a: Real, b: Real) {
    is_derivative_fn(df, ddf)
    implies is_derivative_fn(taylor2_aux_derivative(f, df, a, b),
        taylor2_aux_second_derivative(f, df, ddf, a, b))
} by {
    if is_derivative_fn(df, ddf) {
        forall(x: Real) {
            is_derivative_fn_at(df, ddf, x)
            has_derivative_at(df, x, ddf(x))
            constant_has_derivative_at(df(a), x)
            has_derivative_at(constant[Real, Real](df(a)), x, Real.0)
            derivative_pointwise_sub(df, constant[Real, Real](df(a)), x, ddf(x), Real.0)
            has_derivative_at(pointwise_add(df, pointwise_neg(constant[Real, Real](df(a)))),
                x, ddf(x) + -Real.0)
            neg_zero
            -Real.0 = Real.0
            ddf(x) + -Real.0 = ddf(x) + Real.0
            add_zero_right(ddf(x))
            ddf(x) + Real.0 = ddf(x)
            ddf(x) + -Real.0 = ddf(x)
            has_derivative_at(pointwise_add(df, pointwise_neg(constant[Real, Real](df(a)))),
                x, ddf(x))
            derivative_pointwise_const_mul(square_real(b - a),
                pointwise_add(df, pointwise_neg(constant[Real, Real](df(a)))), x, ddf(x))
            has_derivative_at(pointwise_mul(constant[Real, Real](square_real(b - a)),
                pointwise_add(df, pointwise_neg(constant[Real, Real](df(a))))),
                x, square_real(b - a) * ddf(x))
            affine_real_has_derivative_at(Real.1, -a, x)
            has_derivative_at(affine_real(Real.1, -a), x, Real.1)
            derivative_pointwise_const_mul(taylor2_delta(f, df, a, b) * taylor_two,
                affine_real(Real.1, -a), x, Real.1)
            has_derivative_at(pointwise_mul(constant[Real, Real](taylor2_delta(f, df, a, b) * taylor_two),
                affine_real(Real.1, -a)), x,
                (taylor2_delta(f, df, a, b) * taylor_two) * Real.1)
            mul_one_right(taylor2_delta(f, df, a, b) * taylor_two)
            (taylor2_delta(f, df, a, b) * taylor_two) * Real.1 =
                taylor2_delta(f, df, a, b) * taylor_two
            has_derivative_at(pointwise_mul(constant[Real, Real](taylor2_delta(f, df, a, b) * taylor_two),
                affine_real(Real.1, -a)), x, taylor2_delta(f, df, a, b) * taylor_two)
            derivative_pointwise_sub(
                pointwise_mul(constant[Real, Real](square_real(b - a)),
                    pointwise_add(df, pointwise_neg(constant[Real, Real](df(a))))),
                pointwise_mul(constant[Real, Real](taylor2_delta(f, df, a, b) * taylor_two),
                    affine_real(Real.1, -a)),
                x, square_real(b - a) * ddf(x), taylor2_delta(f, df, a, b) * taylor_two)
            has_derivative_at(pointwise_add(
                pointwise_mul(constant[Real, Real](square_real(b - a)),
                    pointwise_add(df, pointwise_neg(constant[Real, Real](df(a))))),
                pointwise_neg(pointwise_mul(constant[Real, Real](taylor2_delta(f, df, a, b) * taylor_two),
                    affine_real(Real.1, -a)))),
                x, square_real(b - a) * ddf(x) + -(taylor2_delta(f, df, a, b) * taylor_two))
            square_real(b - a) * ddf(x) + -(taylor2_delta(f, df, a, b) * taylor_two) =
                square_real(b - a) * ddf(x) - taylor2_delta(f, df, a, b) * taylor_two
            has_derivative_at(pointwise_add(
                pointwise_mul(constant[Real, Real](square_real(b - a)),
                    pointwise_add(df, pointwise_neg(constant[Real, Real](df(a))))),
                pointwise_neg(pointwise_mul(constant[Real, Real](taylor2_delta(f, df, a, b) * taylor_two),
                    affine_real(Real.1, -a)))),
                x, square_real(b - a) * ddf(x) - taylor2_delta(f, df, a, b) * taylor_two)
            taylor2_aux_second_derivative(f, df, ddf, a, b, x) =
                square_real(b - a) * ddf(x) - taylor2_delta(f, df, a, b) * taylor_two
            taylor2_aux_derivative_eq_pointwise(f, df, a, b)
            taylor2_aux_derivative(f, df, a, b) = pointwise_add(
                pointwise_mul(constant[Real, Real](square_real(b - a)),
                    pointwise_add(df, pointwise_neg(constant[Real, Real](df(a))))),
                pointwise_neg(pointwise_mul(constant[Real, Real](taylor2_delta(f, df, a, b) * taylor_two),
                    affine_real(Real.1, -a))))
            function_eq_transport_predicate_rev(
                function(h: Real -> Real) {
                    has_derivative_at(h, x, taylor2_aux_second_derivative(f, df, ddf, a, b, x))
                },
                taylor2_aux_derivative(f, df, a, b),
                pointwise_add(pointwise_mul(constant[Real, Real](square_real(b - a)),
                        pointwise_add(df, pointwise_neg(constant[Real, Real](df(a))))),
                    pointwise_neg(pointwise_mul(constant[Real, Real](taylor2_delta(f, df, a, b) * taylor_two),
                        affine_real(Real.1, -a)))))
            has_derivative_at(taylor2_aux_derivative(f, df, a, b), x,
                taylor2_aux_second_derivative(f, df, ddf, a, b, x))
        }
        is_derivative_fn_iff(taylor2_aux_derivative(f, df, a, b),
            taylor2_aux_second_derivative(f, df, ddf, a, b))
        is_derivative_fn(taylor2_aux_derivative(f, df, a, b),
            taylor2_aux_second_derivative(f, df, ddf, a, b))
    }
}

/// The auxiliary function of the second-order Taylor theorem vanishes at a.
theorem taylor2_aux_at_a_zero(f: Real -> Real, df: Real -> Real, a: Real, b: Real) {
    taylor2_aux(f, df, a, b, a) = Real.0
} by {
    taylor2_aux(f, df, a, b, a) = square_real(b - a) * taylor2_lin_remainder(f, df, a, a) -
        taylor2_delta(f, df, a, b) * taylor2_sq(a, a)
    taylor2_lin_remainder(f, df, a, a) = f(a) - taylor2_lin(f, df, a, a)
    taylor2_lin(f, df, a, a) = f(a) + df(a) * (a - a)
    a - a = Real.0
    df(a) * (a - a) = df(a) * Real.0
    mul_zero_right(df(a))
    df(a) * Real.0 = Real.0
    add_zero_right(f(a))
    f(a) + Real.0 = f(a)
    taylor2_lin(f, df, a, a) = f(a)
    f(a) - f(a) = Real.0
    taylor2_lin_remainder(f, df, a, a) = Real.0
    mul_zero_right(square_real(b - a))
    square_real(b - a) * Real.0 = Real.0
    square_real(b - a) * taylor2_lin_remainder(f, df, a, a) = Real.0
    taylor2_sq(a, a) = square_real(a - a)
    a - a = Real.0
    square_real(a - a) = square_real(Real.0)
    square_real(Real.0) = Real.0 * Real.0
    mul_zero_left(Real.0)
    Real.0 * Real.0 = Real.0
    square_real(Real.0) = Real.0
    square_real(a - a) = Real.0
    taylor2_sq(a, a) = Real.0
    mul_zero_right(taylor2_delta(f, df, a, b))
    taylor2_delta(f, df, a, b) * Real.0 = Real.0
    taylor2_delta(f, df, a, b) * taylor2_sq(a, a) = Real.0
    Real.0 - Real.0 = Real.0
    taylor2_aux(f, df, a, b, a) = Real.0
}

/// The auxiliary function of the second-order Taylor theorem vanishes at b.
theorem taylor2_aux_at_b_zero(f: Real -> Real, df: Real -> Real, a: Real, b: Real) {
    taylor2_aux(f, df, a, b, b) = Real.0
} by {
    taylor2_aux(f, df, a, b, b) = square_real(b - a) * taylor2_lin_remainder(f, df, a, b) -
        taylor2_delta(f, df, a, b) * taylor2_sq(a, b)
    taylor2_lin_remainder(f, df, a, b) = f(b) - taylor2_lin(f, df, a, b)
    taylor2_lin(f, df, a, b) = f(a) + df(a) * (b - a)
    f(b) - (f(a) + df(a) * (b - a)) = f(b) - f(a) - df(a) * (b - a)
    taylor2_lin_remainder(f, df, a, b) = f(b) - f(a) - df(a) * (b - a)
    taylor2_delta(f, df, a, b) = f(b) - f(a) - df(a) * (b - a)
    taylor2_lin_remainder(f, df, a, b) = taylor2_delta(f, df, a, b)
    square_real(b - a) * taylor2_lin_remainder(f, df, a, b) =
        square_real(b - a) * taylor2_delta(f, df, a, b)
    taylor2_sq(a, b) = square_real(b - a)
    taylor2_delta(f, df, a, b) * taylor2_sq(a, b) =
        taylor2_delta(f, df, a, b) * square_real(b - a)
    taylor2_aux(f, df, a, b, b) =
        square_real(b - a) * taylor2_delta(f, df, a, b) -
            taylor2_delta(f, df, a, b) * square_real(b - a)
    real_mul_comm(square_real(b - a), taylor2_delta(f, df, a, b))
    square_real(b - a) * taylor2_delta(f, df, a, b) =
        taylor2_delta(f, df, a, b) * square_real(b - a)
    square_real(b - a) * taylor2_delta(f, df, a, b) -
        taylor2_delta(f, df, a, b) * square_real(b - a) = Real.0
    taylor2_aux(f, df, a, b, b) = Real.0
}

/// The auxiliary function of the second-order Taylor theorem takes equal
/// values at the endpoints of the interval.
theorem taylor2_aux_endpoints_equal(f: Real -> Real, df: Real -> Real, a: Real, b: Real) {
    taylor2_aux(f, df, a, b, a) = taylor2_aux(f, df, a, b, b)
} by {
    taylor2_aux_at_a_zero(f, df, a, b)
    taylor2_aux(f, df, a, b, a) = Real.0
    taylor2_aux_at_b_zero(f, df, a, b)
    taylor2_aux(f, df, a, b, b) = Real.0
    taylor2_aux(f, df, a, b, a) = taylor2_aux(f, df, a, b, b)
}

/// The derivative of the auxiliary function of the second-order Taylor theorem
/// vanishes at a.
theorem taylor2_aux_derivative_at_a_zero(f: Real -> Real, df: Real -> Real, a: Real, b: Real) {
    taylor2_aux_derivative(f, df, a, b, a) = Real.0
} by {
    taylor2_aux_derivative(f, df, a, b, a) = square_real(b - a) * (df(a) - df(a)) -
        taylor2_delta(f, df, a, b) * taylor_two * (a - a)
    df(a) - df(a) = Real.0
    mul_zero_right(square_real(b - a))
    square_real(b - a) * Real.0 = Real.0
    square_real(b - a) * (df(a) - df(a)) = Real.0
    a - a = Real.0
    mul_zero_right(taylor2_delta(f, df, a, b) * taylor_two)
    (taylor2_delta(f, df, a, b) * taylor_two) * Real.0 = Real.0
    taylor2_delta(f, df, a, b) * taylor_two * (a - a) = Real.0
    Real.0 - Real.0 = Real.0
    taylor2_aux_derivative(f, df, a, b, a) = Real.0
}

/// Subtracting and then adding the same term cancels: (a - b) + b = a.
theorem taylor2_sub_add_cancel(a: Real, b: Real) {
    (a - b) + b = a
} by {
    (a - b) + b = (a + -b) + b
    add_assoc(a, -b, b)
    (a + -b) + b = a + (-b + b)
    add_comm(-b, b)
    -b + b = b + -b
    add_neg_eq_zero(b)
    b + -b = Real.0
    -b + b = Real.0
    a + (-b + b) = a + Real.0
    add_zero_right(a)
    a + Real.0 = a
    (a - b) + b = a
}

/// A double subtraction followed by an addition: (a - b - c) + b = a - c.
theorem taylor2_sub_sub_add(a: Real, b: Real, c: Real) {
    (a - b - c) + b = a - c
} by {
    (a - b - c) + b = ((a - b) + -c) + b
    add_assoc(a - b, -c, b)
    ((a - b) + -c) + b = (a - b) + (-c + b)
    add_comm(-c, b)
    -c + b = b + -c
    (a - b) + (-c + b) = (a - b) + (b + -c)
    add_assoc(a - b, b, -c)
    ((a - b) + b) + -c = (a - b) + (b + -c)
    (a - b) + (b + -c) = ((a - b) + b) + -c
    taylor2_sub_add_cancel(a, b)
    (a - b) + b = a
    ((a - b) + b) + -c = a + -c
    a + -c = a - c
    ((a - b) + b) + -c = a - c
    (a - b - c) + b = a - c
}

/// If the second derivative of the auxiliary function vanishes at c, the
/// second-order Taylor formula holds at b.
theorem taylor2_conclusion_from_second_derivative(f: Real -> Real, df: Real -> Real, ddf: Real -> Real, a: Real, b: Real, c: Real) {
    taylor2_aux_second_derivative(f, df, ddf, a, b, c) = Real.0
    implies f(b) = f(a) + df(a) * (b - a) + (ddf(c) / taylor_two) * (b - a) * (b - a)
} by {
    if taylor2_aux_second_derivative(f, df, ddf, a, b, c) = Real.0 {
        taylor2_aux_second_derivative(f, df, ddf, a, b, c) =
            square_real(b - a) * ddf(c) - taylor2_delta(f, df, a, b) * taylor_two
        square_real(b - a) * ddf(c) - taylor2_delta(f, df, a, b) * taylor_two = Real.0
        sub_zero_imp_eq(square_real(b - a) * ddf(c), taylor2_delta(f, df, a, b) * taylor_two)
        square_real(b - a) * ddf(c) = taylor2_delta(f, df, a, b) * taylor_two
        real_mul_comm(taylor2_delta(f, df, a, b), taylor_two)
        taylor2_delta(f, df, a, b) * taylor_two = taylor_two * taylor2_delta(f, df, a, b)
        square_real(b - a) * ddf(c) = taylor_two * taylor2_delta(f, df, a, b)
        (square_real(b - a) * ddf(c)) * taylor_two.inverse =
            (taylor_two * taylor2_delta(f, df, a, b)) * taylor_two.inverse
        taylor_two_not_zero
        mul_inverse(taylor_two)
        taylor_two * taylor_two.inverse = Real.1
        mul_assoc(taylor_two, taylor2_delta(f, df, a, b), taylor_two.inverse)
        (taylor_two * taylor2_delta(f, df, a, b)) * taylor_two.inverse =
            taylor_two * (taylor2_delta(f, df, a, b) * taylor_two.inverse)
        real_mul_comm(taylor2_delta(f, df, a, b), taylor_two.inverse)
        taylor2_delta(f, df, a, b) * taylor_two.inverse =
            taylor_two.inverse * taylor2_delta(f, df, a, b)
        mul_assoc(taylor_two, taylor_two.inverse, taylor2_delta(f, df, a, b))
        taylor_two * (taylor_two.inverse * taylor2_delta(f, df, a, b)) =
            (taylor_two * taylor_two.inverse) * taylor2_delta(f, df, a, b)
        (taylor_two * taylor_two.inverse) * taylor2_delta(f, df, a, b) =
            Real.1 * taylor2_delta(f, df, a, b)
        mul_one_left(taylor2_delta(f, df, a, b))
        Real.1 * taylor2_delta(f, df, a, b) = taylor2_delta(f, df, a, b)
        taylor_two * (taylor_two.inverse * taylor2_delta(f, df, a, b)) =
            taylor2_delta(f, df, a, b)
        (taylor_two * taylor2_delta(f, df, a, b)) * taylor_two.inverse =
            taylor2_delta(f, df, a, b)
        (square_real(b - a) * ddf(c)) * taylor_two.inverse =
            taylor2_delta(f, df, a, b)
        (square_real(b - a) * ddf(c)) / taylor_two =
            (square_real(b - a) * ddf(c)) * taylor_two.inverse
        (square_real(b - a) * ddf(c)) / taylor_two = taylor2_delta(f, df, a, b)
        taylor2_delta(f, df, a, b) = f(b) - f(a) - df(a) * (b - a)
        (square_real(b - a) * ddf(c)) / taylor_two = f(b) - f(a) - df(a) * (b - a)
        taylor2_sub_sub_add(f(b), f(a), df(a) * (b - a))
        (f(b) - f(a) - df(a) * (b - a)) + f(a) = f(b) - df(a) * (b - a)
        ((square_real(b - a) * ddf(c)) / taylor_two) + f(a) =
            f(b) - df(a) * (b - a)
        (((square_real(b - a) * ddf(c)) / taylor_two) + f(a)) + df(a) * (b - a) =
            (f(b) - df(a) * (b - a)) + df(a) * (b - a)
        taylor2_sub_add_cancel(f(b), df(a) * (b - a))
        (f(b) - df(a) * (b - a)) + df(a) * (b - a) = f(b)
        (((square_real(b - a) * ddf(c)) / taylor_two) + f(a)) + df(a) * (b - a) = f(b)
        add_comm(f(a), df(a) * (b - a))
        f(a) + df(a) * (b - a) = df(a) * (b - a) + f(a)
        add_assoc((square_real(b - a) * ddf(c)) / taylor_two, f(a), df(a) * (b - a))
        (((square_real(b - a) * ddf(c)) / taylor_two) + f(a)) + df(a) * (b - a) =
            ((square_real(b - a) * ddf(c)) / taylor_two) + (f(a) + df(a) * (b - a))
        add_assoc((square_real(b - a) * ddf(c)) / taylor_two, df(a) * (b - a), f(a))
        (((square_real(b - a) * ddf(c)) / taylor_two) + df(a) * (b - a)) + f(a) =
            ((square_real(b - a) * ddf(c)) / taylor_two) + (df(a) * (b - a) + f(a))
        f(a) + df(a) * (b - a) = df(a) * (b - a) + f(a)
        ((square_real(b - a) * ddf(c)) / taylor_two) + (f(a) + df(a) * (b - a)) =
            ((square_real(b - a) * ddf(c)) / taylor_two) + (df(a) * (b - a) + f(a))
        ((square_real(b - a) * ddf(c)) / taylor_two) + (f(a) + df(a) * (b - a)) =
            (((square_real(b - a) * ddf(c)) / taylor_two) + df(a) * (b - a)) + f(a)
        ((square_real(b - a) * ddf(c)) / taylor_two) + (f(a) + df(a) * (b - a)) =
            (((square_real(b - a) * ddf(c)) / taylor_two) + f(a)) + df(a) * (b - a)
        ((square_real(b - a) * ddf(c)) / taylor_two) + (f(a) + df(a) * (b - a)) = f(b)
        f(b) = ((square_real(b - a) * ddf(c)) / taylor_two) + (f(a) + df(a) * (b - a))
        add_comm((square_real(b - a) * ddf(c)) / taylor_two, f(a) + df(a) * (b - a))
        ((square_real(b - a) * ddf(c)) / taylor_two) + (f(a) + df(a) * (b - a)) =
            (f(a) + df(a) * (b - a)) + ((square_real(b - a) * ddf(c)) / taylor_two)
        f(b) = (f(a) + df(a) * (b - a)) + ((square_real(b - a) * ddf(c)) / taylor_two)
        mul_assoc(square_real(b - a), ddf(c), taylor_two.inverse)
        (square_real(b - a) * ddf(c)) * taylor_two.inverse =
            square_real(b - a) * (ddf(c) * taylor_two.inverse)
        (square_real(b - a) * ddf(c)) / taylor_two =
            square_real(b - a) * (ddf(c) * taylor_two.inverse)
        ddf(c) * taylor_two.inverse = ddf(c) / taylor_two
        square_real(b - a) * (ddf(c) * taylor_two.inverse) =
            square_real(b - a) * (ddf(c) / taylor_two)
        (square_real(b - a) * ddf(c)) / taylor_two =
            square_real(b - a) * (ddf(c) / taylor_two)
        f(b) = (f(a) + df(a) * (b - a)) + (square_real(b - a) * (ddf(c) / taylor_two))
        square_real(b - a) = (b - a) * (b - a)
        square_real(b - a) * (ddf(c) / taylor_two) =
            (ddf(c) / taylor_two) * (b - a) * (b - a)
        f(b) = (f(a) + df(a) * (b - a)) + ((ddf(c) / taylor_two) * (b - a) * (b - a))
        f(b) = f(a) + df(a) * (b - a) + ((ddf(c) / taylor_two) * (b - a) * (b - a))
    }
}

/// Taylor's theorem of order two with Lagrange remainder: if f has a second
/// derivative everywhere, then the value at b equals the second-order Taylor
/// polynomial at a plus a remainder f''(c) (b - a)^2 / 2 for some interior c.
theorem taylor_order_two(f: Real -> Real, df: Real -> Real, ddf: Real -> Real, a: Real, b: Real) {
    a < b and is_derivative_fn(f, df) and is_derivative_fn(df, ddf)
    implies exists(c: Real) {
        a < c and c < b and
        f(b) = f(a) + df(a) * (b - a) + (ddf(c) / taylor_two) * (b - a) * (b - a)
    }
} by {
    if a < b and is_derivative_fn(f, df) and is_derivative_fn(df, ddf) {
        taylor2_aux_continuous_on_closed(f, df, a, b, a, b)
        continuous_on_closed(taylor2_aux(f, df, a, b), a, b)
        taylor2_aux_is_derivative_fn(f, df, a, b)
        is_derivative_fn(taylor2_aux(f, df, a, b), taylor2_aux_derivative(f, df, a, b))
        is_derivative_fn_imp_is_derivative_on_open(taylor2_aux(f, df, a, b),
            taylor2_aux_derivative(f, df, a, b), a, b)
        is_derivative_on_open(taylor2_aux(f, df, a, b), taylor2_aux_derivative(f, df, a, b), a, b)
        is_derivative_on_open_imp_differentiable_on_open(taylor2_aux(f, df, a, b),
            taylor2_aux_derivative(f, df, a, b), a, b)
        differentiable_on_open(taylor2_aux(f, df, a, b), a, b)
        taylor2_aux_endpoints_equal(f, df, a, b)
        taylor2_aux(f, df, a, b, a) = taylor2_aux(f, df, a, b, b)
        rolle_theorem(taylor2_aux(f, df, a, b), a, b)
        let c1: Real satisfy {
            a < c1 and c1 < b and has_derivative_at(taylor2_aux(f, df, a, b), c1, Real.0)
        }
        a < c1 and c1 < b
        has_derivative_at(taylor2_aux(f, df, a, b), c1, Real.0)
        is_derivative_fn_at(taylor2_aux(f, df, a, b), taylor2_aux_derivative(f, df, a, b), c1)
        has_derivative_at(taylor2_aux(f, df, a, b), c1, taylor2_aux_derivative(f, df, a, b, c1))
        has_derivative_at_unique(taylor2_aux(f, df, a, b), c1, Real.0,
            taylor2_aux_derivative(f, df, a, b, c1))
        Real.0 = taylor2_aux_derivative(f, df, a, b, c1)
        taylor2_aux_derivative(f, df, a, b, c1) = Real.0
        taylor2_aux_derivative_continuous_on_closed(f, df, ddf, a, b, a, c1)
        continuous_on_closed(taylor2_aux_derivative(f, df, a, b), a, c1)
        taylor2_aux_derivative_is_derivative_fn(f, df, ddf, a, b)
        is_derivative_fn(taylor2_aux_derivative(f, df, a, b),
            taylor2_aux_second_derivative(f, df, ddf, a, b))
        is_derivative_fn_imp_is_derivative_on_open(taylor2_aux_derivative(f, df, a, b),
            taylor2_aux_second_derivative(f, df, ddf, a, b), a, c1)
        is_derivative_on_open(taylor2_aux_derivative(f, df, a, b),
            taylor2_aux_second_derivative(f, df, ddf, a, b), a, c1)
        is_derivative_on_open_imp_differentiable_on_open(taylor2_aux_derivative(f, df, a, b),
            taylor2_aux_second_derivative(f, df, ddf, a, b), a, c1)
        differentiable_on_open(taylor2_aux_derivative(f, df, a, b), a, c1)
        taylor2_aux_derivative_at_a_zero(f, df, a, b)
        taylor2_aux_derivative(f, df, a, b, a) = Real.0
        taylor2_aux_derivative(f, df, a, b, c1) = Real.0
        taylor2_aux_derivative(f, df, a, b, a) = taylor2_aux_derivative(f, df, a, b, c1)
        rolle_theorem(taylor2_aux_derivative(f, df, a, b), a, c1)
        let c2: Real satisfy {
            a < c2 and c2 < c1 and has_derivative_at(taylor2_aux_derivative(f, df, a, b), c2, Real.0)
        }
        a < c2 and c2 < c1
        has_derivative_at(taylor2_aux_derivative(f, df, a, b), c2, Real.0)
        lt_trans[Real](c2, c1, b)
        c2 < b
        is_derivative_fn_at(taylor2_aux_derivative(f, df, a, b),
            taylor2_aux_second_derivative(f, df, ddf, a, b), c2)
        has_derivative_at(taylor2_aux_derivative(f, df, a, b), c2,
            taylor2_aux_second_derivative(f, df, ddf, a, b, c2))
        has_derivative_at_unique(taylor2_aux_derivative(f, df, a, b), c2, Real.0,
            taylor2_aux_second_derivative(f, df, ddf, a, b, c2))
        Real.0 = taylor2_aux_second_derivative(f, df, ddf, a, b, c2)
        taylor2_aux_second_derivative(f, df, ddf, a, b, c2) = Real.0
        taylor2_conclusion_from_second_derivative(f, df, ddf, a, b, c2)
        f(b) = f(a) + df(a) * (b - a) + (ddf(c2) / taylor_two) * (b - a) * (b - a)
        exists(c3: Real) {
            a < c3 and c3 < b and
            f(b) = f(a) + df(a) * (b - a) + (ddf(c3) / taylor_two) * (b - a) * (b - a)
        }
    }
}
