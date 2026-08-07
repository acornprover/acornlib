from data.basic.function_algebra import pointwise_add, pointwise_mul
from data.basic.functions import compose, identity_fn
from real.derivative_basic import has_derivative_at, differentiable_at,
    identity_has_derivative_at, identity_differentiable_at
from real.derivative_chain import derivative_compose, differentiable_compose
from real.derivative_linear import affine_identity_has_derivative_at,
    affine_identity_differentiable_at
from real.real_base import Real

/// Three right-associated compositions satisfy the iterated chain rule.
theorem derivative_compose_three_right(
    f: Real -> Real, g: Real -> Real, h: Real -> Real,
    x0: Real, df: Real, dg: Real, dh: Real
) {
    has_derivative_at(f, x0, df) and
    has_derivative_at(g, f(x0), dg) and
    has_derivative_at(h, g(f(x0)), dh)
    implies has_derivative_at(compose(h, compose(g, f)), x0, dh * (dg * df))
} by {
    derivative_compose(f, g, x0, df, dg)
    has_derivative_at(compose(g, f), x0, dg * df)
    compose(g, f, x0) = g(f(x0))
    derivative_compose(compose(g, f), h, x0, dg * df, dh)
    has_derivative_at(compose(h, compose(g, f)), x0, dh * (dg * df))
}

/// Three left-associated compositions satisfy the iterated chain rule.
theorem derivative_compose_three_left(
    f: Real -> Real, g: Real -> Real, h: Real -> Real,
    x0: Real, df: Real, dg: Real, dh: Real
) {
    has_derivative_at(f, x0, df) and
    has_derivative_at(g, f(x0), dg) and
    has_derivative_at(h, g(f(x0)), dh)
    implies has_derivative_at(compose(compose(h, g), f), x0, (dh * dg) * df)
} by {
    derivative_compose(g, h, f(x0), dg, dh)
    has_derivative_at(compose(h, g), f(x0), dh * dg)
    derivative_compose(f, compose(h, g), x0, df, dh * dg)
    has_derivative_at(compose(compose(h, g), f), x0, (dh * dg) * df)
}

/// Differentiability is preserved by three right-associated compositions.
theorem differentiable_compose_three_right(f: Real -> Real, g: Real -> Real, h: Real -> Real, x0: Real) {
    differentiable_at(f, x0) and differentiable_at(g, f(x0)) and differentiable_at(h, g(f(x0)))
    implies differentiable_at(compose(h, compose(g, f)), x0)
} by {
    differentiable_compose(f, g, x0)
    differentiable_at(compose(g, f), x0)
    compose(g, f, x0) = g(f(x0))
    differentiable_compose(compose(g, f), h, x0)
    differentiable_at(compose(h, compose(g, f)), x0)
}

/// Differentiability is preserved by three left-associated compositions.
theorem differentiable_compose_three_left(f: Real -> Real, g: Real -> Real, h: Real -> Real, x0: Real) {
    differentiable_at(f, x0) and differentiable_at(g, f(x0)) and differentiable_at(h, g(f(x0)))
    implies differentiable_at(compose(compose(h, g), f), x0)
} by {
    differentiable_compose(g, h, f(x0))
    differentiable_at(compose(h, g), f(x0))
    differentiable_compose(f, compose(h, g), x0)
    differentiable_at(compose(compose(h, g), f), x0)
}

/// Four right-associated compositions satisfy the iterated chain rule.
theorem derivative_compose_four_right(
    f: Real -> Real, g: Real -> Real, h: Real -> Real, k: Real -> Real,
    x0: Real, df: Real, dg: Real, dh: Real, dk: Real
) {
    has_derivative_at(f, x0, df) and
    has_derivative_at(g, f(x0), dg) and
    has_derivative_at(h, g(f(x0)), dh) and
    has_derivative_at(k, h(g(f(x0))), dk)
    implies has_derivative_at(compose(k, compose(h, compose(g, f))), x0, dk * (dh * (dg * df)))
} by {
    derivative_compose_three_right(f, g, h, x0, df, dg, dh)
    has_derivative_at(compose(h, compose(g, f)), x0, dh * (dg * df))
    compose(g, f, x0) = g(f(x0))
    compose(h, compose(g, f), x0) = h(g(f(x0)))
    derivative_compose(compose(h, compose(g, f)), k, x0, dh * (dg * df), dk)
    has_derivative_at(compose(k, compose(h, compose(g, f))), x0, dk * (dh * (dg * df)))
}

/// Four left-associated compositions satisfy the iterated chain rule.
theorem derivative_compose_four_left(
    f: Real -> Real, g: Real -> Real, h: Real -> Real, k: Real -> Real,
    x0: Real, df: Real, dg: Real, dh: Real, dk: Real
) {
    has_derivative_at(f, x0, df) and
    has_derivative_at(g, f(x0), dg) and
    has_derivative_at(h, g(f(x0)), dh) and
    has_derivative_at(k, h(g(f(x0))), dk)
    implies has_derivative_at(compose(compose(compose(k, h), g), f), x0, ((dk * dh) * dg) * df)
} by {
    derivative_compose(h, k, g(f(x0)), dh, dk)
    has_derivative_at(compose(k, h), g(f(x0)), dk * dh)
    derivative_compose(g, compose(k, h), f(x0), dg, dk * dh)
    has_derivative_at(compose(compose(k, h), g), f(x0), (dk * dh) * dg)
    derivative_compose(f, compose(compose(k, h), g), x0, df, (dk * dh) * dg)
    has_derivative_at(compose(compose(compose(k, h), g), f), x0, ((dk * dh) * dg) * df)
}

/// Differentiability is preserved by four right-associated compositions.
theorem differentiable_compose_four_right(
    f: Real -> Real, g: Real -> Real, h: Real -> Real, k: Real -> Real, x0: Real
) {
    differentiable_at(f, x0) and differentiable_at(g, f(x0)) and
    differentiable_at(h, g(f(x0))) and differentiable_at(k, h(g(f(x0))))
    implies differentiable_at(compose(k, compose(h, compose(g, f))), x0)
} by {
    differentiable_compose_three_right(f, g, h, x0)
    differentiable_at(compose(h, compose(g, f)), x0)
    compose(g, f, x0) = g(f(x0))
    compose(h, compose(g, f), x0) = h(g(f(x0)))
    differentiable_compose(compose(h, compose(g, f)), k, x0)
    differentiable_at(compose(k, compose(h, compose(g, f))), x0)
}

/// Differentiability is preserved by four left-associated compositions.
theorem differentiable_compose_four_left(
    f: Real -> Real, g: Real -> Real, h: Real -> Real, k: Real -> Real, x0: Real
) {
    differentiable_at(f, x0) and differentiable_at(g, f(x0)) and
    differentiable_at(h, g(f(x0))) and differentiable_at(k, h(g(f(x0))))
    implies differentiable_at(compose(compose(compose(k, h), g), f), x0)
} by {
    differentiable_compose(h, k, g(f(x0)))
    differentiable_at(compose(k, h), g(f(x0)))
    differentiable_compose(g, compose(k, h), f(x0))
    differentiable_at(compose(compose(k, h), g), f(x0))
    differentiable_compose(f, compose(compose(k, h), g), x0)
    differentiable_at(compose(compose(compose(k, h), g), f), x0)
}

/// Composing with the identity as the outer function preserves the derivative.
theorem derivative_compose_identity_outer(f: Real -> Real, x0: Real, d: Real) {
    has_derivative_at(f, x0, d) implies has_derivative_at(compose(identity_fn[Real], f), x0, d)
} by {
    identity_has_derivative_at(f(x0))
    derivative_compose(f, identity_fn[Real], x0, d, Real.1)
    has_derivative_at(compose(identity_fn[Real], f), x0, Real.1 * d)
    Real.1 * d = d
    has_derivative_at(compose(identity_fn[Real], f), x0, d)
}

/// Composing with the identity as the inner function preserves the derivative.
theorem derivative_compose_identity_inner(f: Real -> Real, x0: Real, d: Real) {
    has_derivative_at(f, x0, d) implies has_derivative_at(compose(f, identity_fn[Real]), x0, d)
} by {
    identity_has_derivative_at(x0)
    identity_fn[Real](x0) = x0
    derivative_compose(identity_fn[Real], f, x0, Real.1, d)
    has_derivative_at(compose(f, identity_fn[Real]), x0, d * Real.1)
    d * Real.1 = d
    has_derivative_at(compose(f, identity_fn[Real]), x0, d)
}

/// Differentiability is preserved by composing with the identity as the outer function.
theorem differentiable_compose_identity_outer(f: Real -> Real, x0: Real) {
    differentiable_at(f, x0) implies differentiable_at(compose(identity_fn[Real], f), x0)
} by {
    identity_differentiable_at(f(x0))
    differentiable_compose(f, identity_fn[Real], x0)
    differentiable_at(compose(identity_fn[Real], f), x0)
}

/// Differentiability is preserved by composing with the identity as the inner function.
theorem differentiable_compose_identity_inner(f: Real -> Real, x0: Real) {
    differentiable_at(f, x0) implies differentiable_at(compose(f, identity_fn[Real]), x0)
} by {
    identity_differentiable_at(x0)
    identity_fn[Real](x0) = x0
    differentiable_compose(identity_fn[Real], f, x0)
    differentiable_at(compose(f, identity_fn[Real]), x0)
}

/// A self-composition has derivative given by the chain rule at the two relevant points.
theorem derivative_self_compose(f: Real -> Real, x0: Real, d0: Real, d1: Real) {
    has_derivative_at(f, x0, d0) and has_derivative_at(f, f(x0), d1)
    implies has_derivative_at(compose(f, f), x0, d1 * d0)
} by {
    derivative_compose(f, f, x0, d0, d1)
    has_derivative_at(compose(f, f), x0, d1 * d0)
}

/// Differentiability is preserved by self-composition when both needed pointwise derivatives exist.
theorem differentiable_self_compose(f: Real -> Real, x0: Real) {
    differentiable_at(f, x0) and differentiable_at(f, f(x0))
    implies differentiable_at(compose(f, f), x0)
} by {
    differentiable_compose(f, f, x0)
    differentiable_at(compose(f, f), x0)
}

/// A triple self-composition has derivative given by the right-associated iterated chain rule.
theorem derivative_self_compose_three_right(f: Real -> Real, x0: Real, d0: Real, d1: Real, d2: Real) {
    has_derivative_at(f, x0, d0) and has_derivative_at(f, f(x0), d1) and
    has_derivative_at(f, f(f(x0)), d2)
    implies has_derivative_at(compose(f, compose(f, f)), x0, d2 * (d1 * d0))
} by {
    derivative_compose_three_right(f, f, f, x0, d0, d1, d2)
    has_derivative_at(compose(f, compose(f, f)), x0, d2 * (d1 * d0))
}

/// A triple self-composition has derivative given by the left-associated iterated chain rule.
theorem derivative_self_compose_three_left(f: Real -> Real, x0: Real, d0: Real, d1: Real, d2: Real) {
    has_derivative_at(f, x0, d0) and has_derivative_at(f, f(x0), d1) and
    has_derivative_at(f, f(f(x0)), d2)
    implies has_derivative_at(compose(compose(f, f), f), x0, (d2 * d1) * d0)
} by {
    derivative_compose_three_left(f, f, f, x0, d0, d1, d2)
    has_derivative_at(compose(compose(f, f), f), x0, (d2 * d1) * d0)
}

/// Differentiability is preserved by right-associated triple self-composition.
theorem differentiable_self_compose_three_right(f: Real -> Real, x0: Real) {
    differentiable_at(f, x0) and differentiable_at(f, f(x0)) and differentiable_at(f, f(f(x0)))
    implies differentiable_at(compose(f, compose(f, f)), x0)
} by {
    differentiable_compose_three_right(f, f, f, x0)
    differentiable_at(compose(f, compose(f, f)), x0)
}

/// Differentiability is preserved by left-associated triple self-composition.
theorem differentiable_self_compose_three_left(f: Real -> Real, x0: Real) {
    differentiable_at(f, x0) and differentiable_at(f, f(x0)) and differentiable_at(f, f(f(x0)))
    implies differentiable_at(compose(compose(f, f), f), x0)
} by {
    differentiable_compose_three_left(f, f, f, x0)
    differentiable_at(compose(compose(f, f), f), x0)
}

/// An affine outer function composed with a differentiable function has derivative c times the inner derivative.
theorem derivative_affine_after(
    c: Real, b: Real, f: Real -> Real, x0: Real, d: Real
) {
    has_derivative_at(f, x0, d)
    implies has_derivative_at(
        compose(pointwise_add(pointwise_mul(constant[Real, Real](c), identity_fn[Real]), constant[Real, Real](b)), f),
        x0,
        c * d
    )
} by {
    affine_identity_has_derivative_at(c, b, f(x0))
    derivative_compose(f, pointwise_add(pointwise_mul(constant[Real, Real](c), identity_fn[Real]), constant[Real, Real](b)), x0, d, c)
    has_derivative_at(
        compose(pointwise_add(pointwise_mul(constant[Real, Real](c), identity_fn[Real]), constant[Real, Real](b)), f),
        x0,
        c * d
    )
}

/// Differentiability is preserved by composing an affine function after a differentiable function.
theorem differentiable_affine_after(c: Real, b: Real, f: Real -> Real, x0: Real) {
    differentiable_at(f, x0)
    implies differentiable_at(
        compose(pointwise_add(pointwise_mul(constant[Real, Real](c), identity_fn[Real]), constant[Real, Real](b)), f),
        x0
    )
} by {
    affine_identity_differentiable_at(c, b, f(x0))
    differentiable_compose(f, pointwise_add(pointwise_mul(constant[Real, Real](c), identity_fn[Real]), constant[Real, Real](b)), x0)
    differentiable_at(
        compose(pointwise_add(pointwise_mul(constant[Real, Real](c), identity_fn[Real]), constant[Real, Real](b)), f),
        x0
    )
}

/// Composing a differentiable function after an affine function follows the chain rule.
theorem derivative_after_affine(
    c: Real, b: Real, f: Real -> Real, x0: Real, d: Real
) {
    has_derivative_at(
        f,
        pointwise_add(pointwise_mul(constant[Real, Real](c), identity_fn[Real]), constant[Real, Real](b), x0),
        d
    )
    implies has_derivative_at(
        compose(f, pointwise_add(pointwise_mul(constant[Real, Real](c), identity_fn[Real]), constant[Real, Real](b))),
        x0,
        d * c
    )
} by {
    affine_identity_has_derivative_at(c, b, x0)
    derivative_compose(
        pointwise_add(pointwise_mul(constant[Real, Real](c), identity_fn[Real]), constant[Real, Real](b)),
        f,
        x0,
        c,
        d
    )
    has_derivative_at(
        compose(f, pointwise_add(pointwise_mul(constant[Real, Real](c), identity_fn[Real]), constant[Real, Real](b))),
        x0,
        d * c
    )
}

/// Differentiability is preserved by composing a differentiable function after an affine function.
theorem differentiable_after_affine(c: Real, b: Real, f: Real -> Real, x0: Real) {
    differentiable_at(
        f,
        pointwise_add(pointwise_mul(constant[Real, Real](c), identity_fn[Real]), constant[Real, Real](b), x0)
    )
    implies differentiable_at(
        compose(f, pointwise_add(pointwise_mul(constant[Real, Real](c), identity_fn[Real]), constant[Real, Real](b))),
        x0
    )
} by {
    affine_identity_differentiable_at(c, b, x0)
    differentiable_compose(
        pointwise_add(pointwise_mul(constant[Real, Real](c), identity_fn[Real]), constant[Real, Real](b)),
        f,
        x0
    )
    differentiable_at(
        compose(f, pointwise_add(pointwise_mul(constant[Real, Real](c), identity_fn[Real]), constant[Real, Real](b))),
        x0
    )
}

/// The composition of two affine real functions has derivative equal to the product of slopes.
theorem derivative_affine_after_affine(c: Real, b: Real, a: Real, r: Real, x0: Real) {
    has_derivative_at(
        compose(
            pointwise_add(pointwise_mul(constant[Real, Real](c), identity_fn[Real]), constant[Real, Real](b)),
            pointwise_add(pointwise_mul(constant[Real, Real](a), identity_fn[Real]), constant[Real, Real](r))
        ),
        x0,
        c * a
    )
} by {
    affine_identity_has_derivative_at(a, r, x0)
    affine_identity_has_derivative_at(
        c,
        b,
        pointwise_add(pointwise_mul(constant[Real, Real](a), identity_fn[Real]), constant[Real, Real](r), x0)
    )
    derivative_compose(
        pointwise_add(pointwise_mul(constant[Real, Real](a), identity_fn[Real]), constant[Real, Real](r)),
        pointwise_add(pointwise_mul(constant[Real, Real](c), identity_fn[Real]), constant[Real, Real](b)),
        x0,
        a,
        c
    )
    has_derivative_at(
        compose(
            pointwise_add(pointwise_mul(constant[Real, Real](c), identity_fn[Real]), constant[Real, Real](b)),
            pointwise_add(pointwise_mul(constant[Real, Real](a), identity_fn[Real]), constant[Real, Real](r))
        ),
        x0,
        c * a
    )
}

/// The composition of two affine real functions is differentiable at every point.
theorem differentiable_affine_after_affine(c: Real, b: Real, a: Real, r: Real, x0: Real) {
    differentiable_at(
        compose(
            pointwise_add(pointwise_mul(constant[Real, Real](c), identity_fn[Real]), constant[Real, Real](b)),
            pointwise_add(pointwise_mul(constant[Real, Real](a), identity_fn[Real]), constant[Real, Real](r))
        ),
        x0
    )
} by {
    let affine_a = pointwise_add(pointwise_mul(constant[Real, Real](a), identity_fn[Real]), constant[Real, Real](r))
    let affine_c = pointwise_add(pointwise_mul(constant[Real, Real](c), identity_fn[Real]), constant[Real, Real](b))
    derivative_affine_after_affine(c, b, a, r, x0)
    has_derivative_at(compose(affine_c, affine_a), x0, c * a)
    exists(d: Real) {
        has_derivative_at(compose(affine_c, affine_a), x0, d)
    }
}

/// The right-associated composition of three affine real functions follows the iterated chain rule.
theorem derivative_affine_three_right(
    e: Real, s: Real, c: Real, b: Real, a: Real, r: Real, x0: Real
) {
    has_derivative_at(
        compose(
            pointwise_add(pointwise_mul(constant[Real, Real](e), identity_fn[Real]), constant[Real, Real](s)),
            compose(
                pointwise_add(pointwise_mul(constant[Real, Real](c), identity_fn[Real]), constant[Real, Real](b)),
                pointwise_add(pointwise_mul(constant[Real, Real](a), identity_fn[Real]), constant[Real, Real](r))
            )
        ),
        x0,
        e * (c * a)
    )
} by {
    let affine_a = pointwise_add(pointwise_mul(constant[Real, Real](a), identity_fn[Real]), constant[Real, Real](r))
    let affine_c = pointwise_add(pointwise_mul(constant[Real, Real](c), identity_fn[Real]), constant[Real, Real](b))
    let affine_e = pointwise_add(pointwise_mul(constant[Real, Real](e), identity_fn[Real]), constant[Real, Real](s))
    affine_identity_has_derivative_at(a, r, x0)
    has_derivative_at(affine_a, x0, a)
    affine_identity_has_derivative_at(c, b, affine_a(x0))
    has_derivative_at(affine_c, affine_a(x0), c)
    affine_identity_has_derivative_at(e, s, affine_c(affine_a(x0)))
    has_derivative_at(affine_e, affine_c(affine_a(x0)), e)
    derivative_compose_three_right(affine_a, affine_c, affine_e, x0, a, c, e)
    has_derivative_at(compose(affine_e, compose(affine_c, affine_a)), x0, e * (c * a))
}

/// The right-associated composition of three affine real functions is differentiable at every point.
theorem differentiable_affine_three_right(
    e: Real, s: Real, c: Real, b: Real, a: Real, r: Real, x0: Real
) {
    differentiable_at(
        compose(
            pointwise_add(pointwise_mul(constant[Real, Real](e), identity_fn[Real]), constant[Real, Real](s)),
            compose(
                pointwise_add(pointwise_mul(constant[Real, Real](c), identity_fn[Real]), constant[Real, Real](b)),
                pointwise_add(pointwise_mul(constant[Real, Real](a), identity_fn[Real]), constant[Real, Real](r))
            )
        ),
        x0
    )
} by {
    let affine_a = pointwise_add(pointwise_mul(constant[Real, Real](a), identity_fn[Real]), constant[Real, Real](r))
    let affine_c = pointwise_add(pointwise_mul(constant[Real, Real](c), identity_fn[Real]), constant[Real, Real](b))
    let affine_e = pointwise_add(pointwise_mul(constant[Real, Real](e), identity_fn[Real]), constant[Real, Real](s))
    derivative_affine_three_right(e, s, c, b, a, r, x0)
    has_derivative_at(compose(affine_e, compose(affine_c, affine_a)), x0, e * (c * a))
    exists(d: Real) {
        has_derivative_at(compose(affine_e, compose(affine_c, affine_a)), x0, d)
    }
}
