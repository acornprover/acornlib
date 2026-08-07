/// Derivative bridge for the named real affine function `affine_real`.

from data.basic.function_algebra import pointwise_add, pointwise_mul
from data.basic.functions import compose, identity_fn, function_extensionality,
    function_eq_transport_predicate_rev
from real.continuity_affine import affine_real
from real.derivative_basic import has_derivative_at, differentiable_at
from real.derivative_linear import affine_identity_has_derivative_at,
    affine_identity_differentiable_at
from real.derivative_chain_corollaries import derivative_affine_after,
    differentiable_affine_after
from real.real_base import Real

/// The named affine function agrees with the pointwise affine normal form used by derivative lemmas.
theorem affine_real_eq_pointwise_affine(a: Real, b: Real) {
    affine_real(a, b) =
        pointwise_add(pointwise_mul(constant[Real, Real](a), identity_fn[Real]), constant[Real, Real](b))
} by {
    let p = pointwise_add(pointwise_mul(constant[Real, Real](a), identity_fn[Real]), constant[Real, Real](b))
    forall(x: Real) {
        identity_fn[Real](x) = x
        constant[Real, Real](a, x) = a
        constant[Real, Real](b, x) = b
        pointwise_mul(constant[Real, Real](a), identity_fn[Real], x) = a * x
        p(x) = a * x + b
        affine_real(a, b, x) = a * x + b
        affine_real(a, b, x) = p(x)
    }
    function_extensionality(affine_real(a, b), p)
}

/// The named affine function `x ↦ a * x + b` has derivative `a` at every point.
theorem affine_real_has_derivative_at(a: Real, b: Real, x0: Real) {
    has_derivative_at(affine_real(a, b), x0, a)
} by {
    define derivative_pred(h: Real -> Real) -> Bool {
        has_derivative_at(h, x0, a)
    }
    let p = pointwise_add(pointwise_mul(constant[Real, Real](a), identity_fn[Real]), constant[Real, Real](b))
    affine_identity_has_derivative_at(a, b, x0)
    has_derivative_at(p, x0, a)
    affine_real_eq_pointwise_affine(a, b)
    affine_real(a, b) = p
    derivative_pred(p)
    function_eq_transport_predicate_rev(derivative_pred, affine_real(a, b), p)
    has_derivative_at(affine_real(a, b), x0, a)
}

/// The named affine function is differentiable at every point.
theorem affine_real_differentiable_at(a: Real, b: Real, x0: Real) {
    differentiable_at(affine_real(a, b), x0)
} by {
    define differentiable_pred(h: Real -> Real) -> Bool {
        differentiable_at(h, x0)
    }
    let p = pointwise_add(pointwise_mul(constant[Real, Real](a), identity_fn[Real]), constant[Real, Real](b))
    affine_identity_differentiable_at(a, b, x0)
    differentiable_at(p, x0)
    affine_real_eq_pointwise_affine(a, b)
    affine_real(a, b) = p
    differentiable_pred(p)
    function_eq_transport_predicate_rev(differentiable_pred, affine_real(a, b), p)
    differentiable_at(affine_real(a, b), x0)
}

/// Composing a named affine function after a differentiable function multiplies the derivative by the slope.
theorem derivative_affine_real_after(
    f: Real -> Real, a: Real, b: Real, x0: Real, d: Real
) {
    has_derivative_at(f, x0, d)
        implies has_derivative_at(compose(affine_real(a, b), f), x0, a * d)
} by {
    define derivative_pred(h: Real -> Real) -> Bool {
        has_derivative_at(h, x0, a * d)
    }
    if has_derivative_at(f, x0, d) {
        let p = pointwise_add(pointwise_mul(constant[Real, Real](a), identity_fn[Real]), constant[Real, Real](b))
        derivative_affine_after(a, b, f, x0, d)
        has_derivative_at(compose(p, f), x0, a * d)
        affine_real_eq_pointwise_affine(a, b)
        affine_real(a, b) = p
        compose(affine_real(a, b), f) = compose(p, f)
        derivative_pred(compose(p, f))
        function_eq_transport_predicate_rev(derivative_pred, compose(affine_real(a, b), f), compose(p, f))
        has_derivative_at(compose(affine_real(a, b), f), x0, a * d)
    }
}

/// Differentiability is preserved by composing a named affine function after a differentiable function.
theorem differentiable_affine_real_after(f: Real -> Real, a: Real, b: Real, x0: Real) {
    differentiable_at(f, x0)
        implies differentiable_at(compose(affine_real(a, b), f), x0)
} by {
    define differentiable_pred(h: Real -> Real) -> Bool {
        differentiable_at(h, x0)
    }
    if differentiable_at(f, x0) {
        let p = pointwise_add(pointwise_mul(constant[Real, Real](a), identity_fn[Real]), constant[Real, Real](b))
        differentiable_affine_after(a, b, f, x0)
        differentiable_at(compose(p, f), x0)
        affine_real_eq_pointwise_affine(a, b)
        affine_real(a, b) = p
        compose(affine_real(a, b), f) = compose(p, f)
        differentiable_pred(compose(p, f))
        function_eq_transport_predicate_rev(differentiable_pred, compose(affine_real(a, b), f), compose(p, f))
        differentiable_at(compose(affine_real(a, b), f), x0)
    }
}
