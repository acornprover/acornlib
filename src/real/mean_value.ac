/// The mean value theorem and its classical consequences.
///
/// This file develops the fundamental theorem of differential calculus on the
/// real line: Fermat's theorem on interior extrema, the extreme value theorem
/// for continuous functions on closed intervals, Rolle's theorem, the mean
/// value theorem, and their corollaries (constancy from a zero derivative and
/// monotonicity from a nonnegative derivative).

from order import lte_refl, lte_trans, lt_trans, lt_imp_lte, not_lt_imp_gte, lte_antisymm,
    lt_of_lte_of_lt, lt_of_lt_of_lte, lt_imp_ne, lt_imp_ne_symm, not_lt_self, not_lte_imp_gt
from order_set import closed_interval_set, closed_interval_set_contains_eq,
    closed_interval_set_lower_le, closed_interval_set_le_upper,
    closed_interval_set_contains_lower
from order import closed_interval
from real.continuity_base import Real, continuous, continuous_at, add_real_eps_between
from real.derivative_basic import has_derivative_at, differentiable_at, difference_quotient,
    has_derivative_at_delta, forall_elim, sub_ne_zero_of_ne, has_derivative_at_unique
from real.derivative_rules import derivative_pointwise_neg, derivative_pointwise_sub,
    differentiable_pointwise_sub
from real.real_base import add_comm, close_imp_bounds, lt_add_pos, pos_imp_eq_abs,
    neg_zero, neg_distrib, add_assoc,
    self_close, sub_cancels, neg_neg, abs_neg, lt_add_right, lte_add_right, add_neg_eq_zero,
    gt_zero_imp_pos, add_zero_left, add_zero_right, neg_pos_is_neg, neg_lt_zero,
    pos_gt_zero, lt_add_converse, bounds_imp_close, lte_abs, abs_gte_zero, lte_lt_trans
from real.real_seq import eps_smaller_than_both,
    sub_zero_imp_eq, lt_imp_minus_pos, neg_is_close
from real.real_ring import mul_zero_left, mul_zero_right,
    pos_lte_imp_pos, real_mul_comm, mul_distrib_right, mul_neg_left, mul_neg_right
from ordered_field import inverse_of_positive_is_positive, inverse_of_negative_is_negative,
    mul_le_mul_of_nonpos_right, multiply_inequality_with_nonnegative_element
from real.continuity_composition import continuous_imp_continuous_at
from real.continuity_local_bounded import continuous_at_local_abs_bound
from real.continuity_const_sub import const_sub_left, continuous_at_const_sub_left
from real.continuity_affine import affine_real, continuous_at_affine_real
from real.derivative_affine_named import affine_real_has_derivative_at
from real.derivative_continuity import div_mul_cancel_denominator
from real.calculus_api import is_derivative_fn, is_derivative_fn_at
from real.continuity_reciprocal_div import continuous_at_pointwise_reciprocal_real
from real.derivative_quotient import pointwise_reciprocal_real
from real.continuity_pointwise import continuous_at_pointwise_neg, continuous_at_pointwise_add
from data.basic.function_algebra import pointwise_neg, pointwise_add
from data.basic.set import Set, set_image, maps_into_set_image, set_image_contains_witness
from data.basic.functions import function_extensionality, function_eq_transport_predicate_rev,
    identity_fn
from real.supremum import completeness, has_upper_bound, is_nonempty, is_set_supremum,
    is_set_upper_bound, set_member_le_supremum, set_supremum_le_upper_bound,
    set_bound_below_supremum_not_upper, set_not_upper_bound_witness,
    set_supremum_close_from_below, function_image
from real.harmonic import real_inverse_antitone_pos_strict
from real.real_field import mul_inverse, mul_left_cancel,
    zero_is_different_than_one

numerals Real

/// True if f is differentiable at every point of the open interval (lower, upper).
define differentiable_on_open(f: Real -> Real, lower: Real, upper: Real) -> Bool {
    forall(x: Real) {
        lower < x and x < upper implies differentiable_at(f, x)
    }
}

/// True if df is a pointwise derivative of f on the open interval (lower, upper).
define is_derivative_on_open(f: Real -> Real, df: Real -> Real, lower: Real, upper: Real) -> Bool {
    forall(x: Real) {
        lower < x and x < upper implies has_derivative_at(f, x, df(x))
    }
}

/// True if f is continuous at every point of the closed interval [lower, upper].
define continuous_on_closed(f: Real -> Real, lower: Real, upper: Real) -> Bool {
    forall(x: Real) {
        closed_interval_set(lower, upper).contains(x) implies continuous_at(f, x)
    }
}

/// A nonpositive numerator divided by a positive denominator is nonpositive.
theorem le_zero_div_pos_le_zero(a: Real, b: Real) {
    a <= Real.0 and b.is_positive implies a / b <= Real.0
} by {
    if a <= Real.0 and b.is_positive {
        pos_gt_zero(b)
        b > Real.0
        inverse_of_positive_is_positive[Real](b)
        Real.0 < b.inverse
        multiply_inequality_with_nonnegative_element[Real](a, Real.0, b.inverse)
        a * b.inverse <= Real.0 * b.inverse
        mul_zero_left(b.inverse)
        Real.0 * b.inverse = Real.0
        a * b.inverse <= Real.0
        a / b = a * b.inverse
        a / b <= Real.0
    }
}

/// A nonpositive numerator divided by a negative denominator is nonnegative.
theorem le_zero_div_neg_ge_zero(a: Real, b: Real) {
    a <= Real.0 and b.is_negative implies Real.0 <= a / b
} by {
    if a <= Real.0 and b.is_negative {
        neg_lt_zero(b)
        b < Real.0
        inverse_of_negative_is_negative[Real](b)
        b.inverse < Real.0
        lt_imp_lte(b.inverse, Real.0)
        b.inverse <= Real.0
        mul_le_mul_of_nonpos_right[Real](a, Real.0, b.inverse)
        Real.0 * b.inverse <= a * b.inverse
        mul_zero_left(b.inverse)
        Real.0 * b.inverse = Real.0
        Real.0 <= a * b.inverse
        a / b = a * b.inverse
        Real.0 <= a / b
    }
}

/// A real number smaller than every positive tolerance is nonpositive.
theorem lt_all_eps_imp_lte_zero(d: Real) {
    (forall(eps: Real) { eps.is_positive implies d < eps }) implies d <= Real.0
} by {
    if forall(eps: Real) { eps.is_positive implies d < eps } {
        if not d <= Real.0 {
            not_lte_imp_gt[Real](d, Real.0)
            Real.0 < d
            gt_zero_imp_pos(d)
            d.is_positive
            forall(eps: Real) { eps.is_positive implies d < eps }
            d.is_positive implies d < d
            d < d
            not_lt_self(d)
            false
        }
        d <= Real.0
    }
}

/// A value close to a nonpositive number is bounded above by the tolerance.
theorem close_le_zero_imp_lt_eps(q: Real, d: Real, eps: Real) {
    q.is_close(d, eps) and q <= Real.0 implies d < eps
} by {
    if q.is_close(d, eps) and q <= Real.0 {
        close_imp_bounds(q, d, eps)
        d < q + eps
        lte_add_right(q, Real.0, eps)
        q + eps <= Real.0 + eps
        add_zero_left(eps)
        Real.0 + eps = eps
        q + eps <= eps
        lt_of_lt_of_lte[Real](d, q + eps, eps)
        d < eps
    }
}

/// A value close to a nonnegative number has a negative bounded above by the tolerance.
theorem close_ge_zero_imp_neg_lt_eps(q: Real, d: Real, eps: Real) {
    q.is_close(d, eps) and Real.0 <= q implies -d < eps
} by {
    if q.is_close(d, eps) and Real.0 <= q {
        neg_is_close(q, d, eps)
        (-q).is_close(-d, eps)
        if Real.0 <= q {
            lte_add_right(Real.0, q, -q)
            Real.0 + -q <= q + -q
            add_zero_left(-q)
            Real.0 + -q = -q
            add_neg_eq_zero(q)
            q + -q = Real.0
            -q <= Real.0
        }
        close_le_zero_imp_lt_eps(-q, -d, eps)
        -d < eps
    }
}

/// Negating a nonpositive number gives a nonnegative number.
theorem neg_le_zero_imp_ge_zero(x: Real) {
    -x <= Real.0 implies Real.0 <= x
} by {
    if -x <= Real.0 {
        lte_add_right(-x, Real.0, x)
        -x + x <= Real.0 + x
        add_comm(-x, x)
        -x + x = x + -x
        add_neg_eq_zero(x)
        x + -x = Real.0
        -x + x = Real.0
        add_zero_left(x)
        Real.0 + x = x
        Real.0 <= x
    }
}

/// A difference of an ordered pair is nonpositive.
theorem lte_imp_sub_le_zero(u: Real, v: Real) {
    u <= v implies u - v <= Real.0
} by {
    if u <= v {
        lte_add_right(u, v, -v)
        u + -v <= v + -v
        u - v = u + -v
        add_neg_eq_zero(v)
        v + -v = Real.0
        u - v <= Real.0
    }
}

/// A difference of an ordered pair is nonnegative.
theorem lte_imp_zero_le_sub(u: Real, v: Real) {
    u <= v implies Real.0 <= v - u
} by {
    if u <= v {
        lte_add_right(u, v, -u)
        u + -u <= v + -u
        add_neg_eq_zero(u)
        u + -u = Real.0
        v - u = v + -u
        Real.0 <= v - u
    }
}

/// Fermat's theorem: a differentiable function has zero derivative at an
/// interior point where it attains a maximum on the open interval.
theorem fermat_interior_maximum(f: Real -> Real, a: Real, b: Real, x0: Real, d: Real) {
    a < x0 and x0 < b and
    (forall(y: Real) { a < y and y < b implies f(y) <= f(x0) }) and
    has_derivative_at(f, x0, d)
    implies d = Real.0
} by {
    if a < x0 and x0 < b and
       (forall(y: Real) { a < y and y < b implies f(y) <= f(x0) }) and
       has_derivative_at(f, x0, d) {
        // Right difference quotients are nonpositive, so d <= 0.
        forall(eps: Real) {
            if eps.is_positive {
                has_derivative_at_delta(f, x0, d, eps)
                let delta: Real satisfy {
                    delta.is_positive and forall(x: Real) {
                        x != x0 and x.is_close(x0, delta)
                        implies difference_quotient(f, x0, x).is_close(d, eps)
                    }
                }
                lt_imp_minus_pos(x0, b)
                (b - x0).is_positive
                eps_smaller_than_both(delta, b - x0)
                let h: Real satisfy {
                    h.is_positive and h < delta and h < b - x0
                }
                h.is_positive
                let x = x0 + h
                lt_add_pos(x0, h)
                x0 < x
                lt_trans[Real](a, x0, x)
                a < x
                lt_add_right(h, b - x0, x0)
                h + x0 < (b - x0) + x0
                (b - x0) + x0 = b
                h + x0 < b
                add_comm(h, x0)
                h + x0 = x
                x < b
                forall(y: Real) { a < y and y < b implies f(y) <= f(x0) }
                a < x and x < b implies f(x) <= f(x0)
                f(x) <= f(x0)
                lte_imp_sub_le_zero(f(x), f(x0))
                f(x) - f(x0) <= Real.0
                lt_imp_ne_symm(x0, x)
                x != x0
                x - x0 = (x0 + h) - x0
                add_comm(x0, h)
                x0 + h = h + x0
                (x0 + h) - x0 = (h + x0) - x0
                sub_cancels(h, x0)
                h + x0 - x0 = h
                (x0 + h) - x0 = h
                x - x0 = h
                pos_imp_eq_abs(h)
                h = h.abs
                (x - x0).abs = h.abs
                (x - x0).abs = h
                h < delta
                (x - x0).abs < delta
                x.is_close(x0, delta)
                x != x0 and x.is_close(x0, delta)
                forall_elim[Real](function(y: Real) {
                    y != x0 and y.is_close(x0, delta) implies
                        difference_quotient(f, x0, y).is_close(d, eps)
                }, x)
                function(y: Real) {
                    y != x0 and y.is_close(x0, delta) implies
                        difference_quotient(f, x0, y).is_close(d, eps)
                }(x)
                difference_quotient(f, x0, x).is_close(d, eps)
                difference_quotient(f, x0, x) = (f(x) - f(x0)) / (x - x0)
                x - x0 = h
                difference_quotient(f, x0, x) = (f(x) - f(x0)) / h
                h.is_positive
                le_zero_div_pos_le_zero(f(x) - f(x0), h)
                difference_quotient(f, x0, x) <= Real.0
                close_le_zero_imp_lt_eps(difference_quotient(f, x0, x), d, eps)
                d < eps
            }
        }
        lt_all_eps_imp_lte_zero(d)
        d <= Real.0

        // Left difference quotients are nonnegative, so -d <= 0.
        forall(eps: Real) {
            if eps.is_positive {
                has_derivative_at_delta(f, x0, d, eps)
                let delta: Real satisfy {
                    delta.is_positive and forall(x: Real) {
                        x != x0 and x.is_close(x0, delta)
                        implies difference_quotient(f, x0, x).is_close(d, eps)
                    }
                }
                lt_imp_minus_pos(a, x0)
                (x0 - a).is_positive
                eps_smaller_than_both(delta, x0 - a)
                let h: Real satisfy {
                    h.is_positive and h < delta and h < x0 - a
                }
                h.is_positive
                let x = x0 - h
                lt_add_right(h, x0 - a, a)
                h + a < (x0 - a) + a
                (x0 - a) + a = x0
                h + a < x0
                add_comm(h, a)
                h + a = a + h
                a + h < x0
                (x0 - h) + h = x0
                a + h < (x0 - h) + h
                lt_add_converse(a, x0 - h, h)
                a < x
                neg_pos_is_neg(h)
                (-h).is_negative
                neg_lt_zero(-h)
                -h < Real.0
                lt_add_right(-h, Real.0, x0)
                -h + x0 < Real.0 + x0
                add_comm(-h, x0)
                -h + x0 = x0 + -h
                x0 - h = x0 + -h
                x0 + -h < x0
                x0 - h < x0
                x < x0
                lt_trans[Real](x, x0, b)
                x < b
                a < x and x < b
                forall(y: Real) { a < y and y < b implies f(y) <= f(x0) }
                a < x and x < b implies f(x) <= f(x0)
                f(x) <= f(x0)
                lte_imp_sub_le_zero(f(x), f(x0))
                f(x) - f(x0) <= Real.0
                lt_imp_ne(x, x0)
                x != x0
                x - x0 = (x0 - h) - x0
                (x0 - h) - x0 = -h
                x - x0 = -h
                abs_neg(h)
                (-h).abs = h.abs
                pos_imp_eq_abs(h)
                h = h.abs
                (x - x0).abs = (-h).abs
                (x - x0).abs = h
                h < delta
                (x - x0).abs < delta
                x.is_close(x0, delta)
                x != x0 and x.is_close(x0, delta)
                forall_elim[Real](function(y: Real) {
                    y != x0 and y.is_close(x0, delta) implies
                        difference_quotient(f, x0, y).is_close(d, eps)
                }, x)
                function(y: Real) {
                    y != x0 and y.is_close(x0, delta) implies
                        difference_quotient(f, x0, y).is_close(d, eps)
                }(x)
                difference_quotient(f, x0, x).is_close(d, eps)
                difference_quotient(f, x0, x) = (f(x) - f(x0)) / (x - x0)
                x - x0 = -h
                difference_quotient(f, x0, x) = (f(x) - f(x0)) / (-h)
                neg_pos_is_neg(h)
                (-h).is_negative
                le_zero_div_neg_ge_zero(f(x) - f(x0), -h)
                Real.0 <= difference_quotient(f, x0, x)
                close_ge_zero_imp_neg_lt_eps(difference_quotient(f, x0, x), d, eps)
                -d < eps
            }
        }
        lt_all_eps_imp_lte_zero(-d)
        -d <= Real.0
        neg_le_zero_imp_ge_zero(d)
        Real.0 <= d
        lte_antisymm[Real](d, Real.0)
        d = Real.0
    }
}

/// A positive real plus one is positive.
theorem pos_add_one_is_pos(x: Real) {
    x.is_positive implies (x + Real.1).is_positive
} by {
    if x.is_positive {
        pos_gt_zero(x)
        Real.0 < x
        lt_imp_lte(Real.0, x)
        Real.0 <= x
        Real.1.is_positive
        lte_add_right(Real.0, x, Real.1)
        Real.0 + Real.1 <= x + Real.1
        add_zero_left(Real.1)
        Real.0 + Real.1 = Real.1
        Real.1 <= x + Real.1
        pos_lte_imp_pos(Real.1, x + Real.1)
        (x + Real.1).is_positive
    }
}

/// An absolute value plus one is positive.
theorem abs_add_one_is_pos(x: Real) {
    (x.abs + Real.1).is_positive
} by {
    abs_gte_zero(x)
    x.abs >= Real.0
    lte_add_right(Real.0, x.abs, Real.1)
    Real.0 + Real.1 <= x.abs + Real.1
    add_zero_left(Real.1)
    Real.0 + Real.1 = Real.1
    Real.1 <= x.abs + Real.1
    pos_lte_imp_pos(Real.1, x.abs + Real.1)
    (x.abs + Real.1).is_positive
}

/// True if f is bounded above on the closed interval [lower, x].
define bounded_above_on_prefix(f: Real -> Real, lower: Real, x: Real) -> Bool {
    exists(bound: Real) {
        forall(y: Real) {
            closed_interval_set(lower, x).contains(y) implies f(y) <= bound
        }
    }
}

/// Membership in the set of points of [a, b] on whose prefix f is bounded above.
define bounded_prefix_contains(f: Real -> Real, a: Real, b: Real, x: Real) -> Bool {
    a <= x and x <= b and bounded_above_on_prefix(f, a, x)
}

/// The set of points of [a, b] on whose prefix f is bounded above.
define bounded_prefix_set(f: Real -> Real, a: Real, b: Real) -> Set[Real] {
    Set[Real].new(bounded_prefix_contains(f, a, b))
}

/// Membership in the bounded-prefix set is the defining conjunction.
theorem bounded_prefix_set_contains_eq(f: Real -> Real, a: Real, b: Real, x: Real) {
    bounded_prefix_set(f, a, b).contains(x) = bounded_prefix_contains(f, a, b, x)
}

/// The lower endpoint is a member of the bounded-prefix set.
theorem bounded_prefix_contains_lower(f: Real -> Real, a: Real, b: Real) {
    a <= b implies bounded_prefix_set(f, a, b).contains(a)
} by {
    if a <= b {
        lte_refl(a)
        let bound: Real = f(a)
        forall(y: Real) {
            if closed_interval_set(a, a).contains(y) {
                closed_interval_set_contains_eq(a, a, y)
                closed_interval(a, a, y)
                closed_interval(a, a, y) = (a <= y and y <= a)
                a <= y and y <= a
                lte_antisymm[Real](a, y)
                a = y
                y = a
                f(y) = f(a)
                lte_refl(f(a))
                f(a) <= f(a)
                f(y) <= f(a)
                f(y) <= bound
            }
        }
        bounded_above_on_prefix(f, a, a) = exists(bound2: Real) {
            forall(y: Real) {
                closed_interval_set(a, a).contains(y) implies f(y) <= bound2
            }
        }
        exists(bound2: Real) {
            forall(y: Real) {
                closed_interval_set(a, a).contains(y) implies f(y) <= bound2
            }
        }
        bounded_above_on_prefix(f, a, a)
        bounded_prefix_contains(f, a, b, a)
        bounded_prefix_set_contains_eq(f, a, b, a)
        bounded_prefix_set(f, a, b).contains(a)
    }
}

/// The bounded-prefix set is nonempty.
theorem bounded_prefix_set_is_nonempty(f: Real -> Real, a: Real, b: Real) {
    a <= b implies is_nonempty(bounded_prefix_set(f, a, b))
} by {
    if a <= b {
        bounded_prefix_contains_lower(f, a, b)
        let s = bounded_prefix_set(f, a, b)
        s.contains(a)
        exists(x: Real) {
            s.contains(x)
        }
        is_nonempty(s)
    }
}

/// The upper endpoint bounds the bounded-prefix set.
theorem bounded_prefix_set_upper_bound(f: Real -> Real, a: Real, b: Real) {
    is_set_upper_bound(bounded_prefix_set(f, a, b), b)
} by {
    let s = bounded_prefix_set(f, a, b)
    forall(x: Real) {
        if s.contains(x) {
            bounded_prefix_set_contains_eq(f, a, b, x)
            bounded_prefix_contains(f, a, b, x)
            bounded_prefix_contains(f, a, b, x) =
                (a <= x and x <= b and bounded_above_on_prefix(f, a, x))
            x <= b
        }
    }
    function(s0: Set[Real], bound: Real) {
        forall(x0: Real) {
            s0.contains(x0) implies x0 <= bound
        } = is_set_upper_bound(s0, bound)
    }(s, b)
    is_set_upper_bound(s, b)
}

/// The bounded-prefix set has an upper bound.
theorem bounded_prefix_set_has_upper_bound(f: Real -> Real, a: Real, b: Real) {
    has_upper_bound(bounded_prefix_set(f, a, b))
} by {
    bounded_prefix_set_upper_bound(f, a, b)
    let s = bounded_prefix_set(f, a, b)
    exists(bound: Real) {
        is_set_upper_bound(s, bound)
    }
    has_upper_bound(s)
}

/// The bounded-prefix set has a supremum.
theorem bounded_prefix_set_supremum_exists(f: Real -> Real, a: Real, b: Real) {
    a <= b implies exists(t: Real) {
        is_set_supremum(bounded_prefix_set(f, a, b), t)
    }
} by {
    if a <= b {
        let s = bounded_prefix_set(f, a, b)
        bounded_prefix_set_is_nonempty(f, a, b)
        is_nonempty(s)
        bounded_prefix_set_has_upper_bound(f, a, b)
        has_upper_bound(s)
        completeness(s)
        let t: Real satisfy {
            is_set_supremum(s, t)
        }
        is_set_supremum(bounded_prefix_set(f, a, b), t)
        exists(t2: Real) {
            is_set_supremum(bounded_prefix_set(f, a, b), t2)
        }
    }
}

/// A member of the bounded-prefix set lies in the ambient interval.
theorem bounded_prefix_member_in_interval(f: Real -> Real, a: Real, b: Real, x: Real) {
    bounded_prefix_set(f, a, b).contains(x) implies a <= x and x <= b
} by {
    if bounded_prefix_set(f, a, b).contains(x) {
        bounded_prefix_set_contains_eq(f, a, b, x)
        bounded_prefix_contains(f, a, b, x)
        bounded_prefix_contains(f, a, b, x) =
            (a <= x and x <= b and bounded_above_on_prefix(f, a, x))
        a <= x and x <= b and bounded_above_on_prefix(f, a, x)
        a <= x and x <= b
    }
}

/// A member of the bounded-prefix set has a bound on its prefix.
theorem bounded_prefix_member_bound(f: Real -> Real, a: Real, b: Real, x: Real) {
    bounded_prefix_set(f, a, b).contains(x) implies exists(bound: Real) {
        forall(y: Real) {
            closed_interval_set(a, x).contains(y) implies f(y) <= bound
        }
    }
} by {
    if bounded_prefix_set(f, a, b).contains(x) {
        bounded_prefix_set_contains_eq(f, a, b, x)
        bounded_prefix_contains(f, a, b, x)
        bounded_prefix_contains(f, a, b, x) =
            (a <= x and x <= b and bounded_above_on_prefix(f, a, x))
        a <= x and x <= b and bounded_above_on_prefix(f, a, x)
        bounded_above_on_prefix(f, a, x)
        bounded_above_on_prefix(f, a, x) = exists(bound: Real) {
            forall(y: Real) {
                closed_interval_set(a, x).contains(y) implies f(y) <= bound
            }
        }
        let bound: Real satisfy {
            forall(y: Real) {
                closed_interval_set(a, x).contains(y) implies f(y) <= bound
            }
        }
        exists(bound2: Real) {
            forall(y: Real) {
                closed_interval_set(a, x).contains(y) implies f(y) <= bound2
            }
        }
    }
}

/// A function continuous on a closed interval is bounded above on it.
theorem continuous_closed_interval_bounded_above(f: Real -> Real, a: Real, b: Real) {
    continuous_on_closed(f, a, b) and a < b
    implies exists(bound: Real) {
        forall(y: Real) {
            closed_interval_set(a, b).contains(y) implies f(y) <= bound
        }
    }
} by {
    if continuous_on_closed(f, a, b) and a < b {
        lt_imp_lte(a, b)
        a <= b
        bounded_prefix_set_supremum_exists(f, a, b)
        let s = bounded_prefix_set(f, a, b)
        let t: Real satisfy {
            is_set_supremum(s, t)
        }
        is_set_supremum(s, t)
        // Part A: t = b, by extending past any t < b.
        bounded_prefix_set_upper_bound(f, a, b)
        is_set_upper_bound(s, b)
        set_supremum_le_upper_bound(s, t, b)
        t <= b
        if t < b {
            s = bounded_prefix_set(f, a, b)
            bounded_prefix_contains_lower(f, a, b)
            s.contains(a)
            set_member_le_supremum(s, t, a)
            a <= t
            closed_interval(a, b, t)
            closed_interval_set_contains_eq(a, b, t)
            closed_interval_set(a, b).contains(t)
            continuous_on_closed(f, a, b) = forall(x: Real) {
                closed_interval_set(a, b).contains(x) implies continuous_at(f, x)
            }
            closed_interval_set(a, b).contains(t) implies continuous_at(f, t)
            continuous_at(f, t)
            continuous_at_local_abs_bound(f, t)
            let (delta0: Real, bound0: Real) satisfy {
                delta0.is_positive and bound0.is_positive and forall(y: Real) {
                    y.is_close(t, delta0) implies f(y).abs <= bound0
                }
            }
            add_real_eps_between(t, b)
            let eps: Real satisfy {
                eps.is_positive and t + eps < b
            }
            eps_smaller_than_both(eps, delta0)
            let eta: Real satisfy {
                eta.is_positive and eta < eps and eta < delta0
            }
            eta.is_positive
            lt_add_pos(t, eta)
            t < t + eta
            lt_add_right(eta, eps, t)
            eta + t < eps + t
            add_comm(eta, t)
            eta + t = t + eta
            add_comm(eps, t)
            eps + t = t + eps
            t + eta < t + eps
            lt_trans[Real](t + eta, t + eps, b)
            t + eta < b
            lt_imp_lte(t + eta, b)
            t + eta <= b
            bounded_prefix_contains_lower(f, a, b)
            s.contains(a)
            set_member_le_supremum(s, t, a)
            a <= t
            lt_of_lte_of_lt[Real](a, t, t + eta)
            a < t + eta
            lt_imp_lte(a, t + eta)
            a <= t + eta
            // Pick a member s of s with t - delta0 < s.
            neg_pos_is_neg(delta0)
            (-delta0).is_negative
            neg_lt_zero(-delta0)
            -delta0 < Real.0
            lt_add_right(-delta0, Real.0, t)
            -delta0 + t < Real.0 + t
            add_comm(-delta0, t)
            -delta0 + t = t + -delta0
            add_zero_left(t)
            Real.0 + t = t
            t + -delta0 < t
            t - delta0 = t + -delta0
            t - delta0 < t
            set_bound_below_supremum_not_upper(s, t, t - delta0)
            not is_set_upper_bound(s, t - delta0)
            set_not_upper_bound_witness(s, t - delta0)
            let sx: Real satisfy {
                s.contains(sx) and not sx <= t - delta0
            }
            s.contains(sx)
            not_lte_imp_gt[Real](sx, t - delta0)
            t - delta0 < sx
            set_member_le_supremum(s, t, sx)
            sx <= t
            s = bounded_prefix_set(f, a, b)
            bounded_prefix_set(f, a, b).contains(sx)
            bounded_prefix_member_in_interval(f, a, b, sx)
            a <= sx and sx <= b
            bounded_prefix_member_bound(f, a, b, sx)
            let m_s: Real satisfy {
                forall(y: Real) {
                    closed_interval_set(a, sx).contains(y) implies f(y) <= m_s
                }
            }
            // t + eta is a member of s.
            let bound = m_s.abs + bound0 + Real.1
            forall(y: Real) {
                if closed_interval_set(a, t + eta).contains(y) {
                    closed_interval_set_lower_le(a, t + eta, y)
                    a <= y
                    closed_interval_set_le_upper(a, t + eta, y)
                    y <= t + eta
                    if y <= sx {
                        closed_interval(a, sx, y)
                        closed_interval_set_contains_eq(a, sx, y)
                        closed_interval_set(a, sx).contains(y)
                        forall(y0: Real) {
                            closed_interval_set(a, sx).contains(y0) implies f(y0) <= m_s
                        }
                        f(y) <= m_s
                        lte_abs(m_s)
                        m_s <= m_s.abs
                        abs_gte_zero(m_s)
                        m_s.abs >= Real.0
                        bound0.is_positive
                        pos_add_one_is_pos(bound0)
                        (bound0 + Real.1).is_positive
                        lt_add_pos(m_s.abs, bound0 + Real.1)
                        m_s.abs < m_s.abs + (bound0 + Real.1)
                        m_s.abs + (bound0 + Real.1) = m_s.abs + bound0 + Real.1
                        m_s.abs < m_s.abs + bound0 + Real.1
                        lt_imp_lte(m_s.abs, m_s.abs + bound0 + Real.1)
                        m_s.abs <= m_s.abs + bound0 + Real.1
                        lte_trans[Real](m_s, m_s.abs, m_s.abs + bound0 + Real.1)
                        m_s <= m_s.abs + bound0 + Real.1
                        lte_trans[Real](f(y), m_s, m_s.abs + bound0 + Real.1)
                        f(y) <= m_s.abs + bound0 + Real.1
                        f(y) <= bound
                    } else {
                        not y <= sx
                        not_lte_imp_gt[Real](sx, y)
                        sx < y
                        lt_trans[Real](t - delta0, sx, y)
                        t - delta0 < y
                        lt_add_right(eta, delta0, t)
                        eta + t < delta0 + t
                        add_comm(eta, t)
                        eta + t = t + eta
                        add_comm(delta0, t)
                        delta0 + t = t + delta0
                        t + eta < t + delta0
                        lt_of_lte_of_lt[Real](y, t + eta, t + delta0)
                        y < t + delta0
                        bounds_imp_close(y, t, delta0)
                        y.is_close(t, delta0)
                        forall(y0: Real) {
                            y0.is_close(t, delta0) implies f(y0).abs <= bound0
                        }
                        y.is_close(t, delta0) implies f(y).abs <= bound0
                        f(y).abs <= bound0
                        lte_abs(f(y))
                        f(y) <= f(y).abs
                        lte_trans[Real](f(y), f(y).abs, bound0)
                        f(y) <= bound0
                        abs_add_one_is_pos(m_s)
                        (m_s.abs + Real.1).is_positive
                        lt_add_pos(bound0, m_s.abs + Real.1)
                        bound0 < bound0 + (m_s.abs + Real.1)
                        bound0 + (m_s.abs + Real.1) = bound0 + m_s.abs + Real.1
                        bound0 < bound0 + m_s.abs + Real.1
                        add_comm(bound0, m_s.abs)
                        bound0 + m_s.abs = m_s.abs + bound0
                        bound0 + m_s.abs + Real.1 = m_s.abs + bound0 + Real.1
                        bound0 < m_s.abs + bound0 + Real.1
                        lt_imp_lte(bound0, m_s.abs + bound0 + Real.1)
                        bound0 <= m_s.abs + bound0 + Real.1
                        lte_trans[Real](f(y), bound0, m_s.abs + bound0 + Real.1)
                        f(y) <= m_s.abs + bound0 + Real.1
                        f(y) <= bound
                    }
                }
            }
            bounded_above_on_prefix(f, a, t + eta) = exists(bound4: Real) {
                forall(y: Real) {
                    closed_interval_set(a, t + eta).contains(y) implies f(y) <= bound4
                }
            }
            exists(bound4: Real) {
                forall(y: Real) {
                    closed_interval_set(a, t + eta).contains(y) implies f(y) <= bound4
                }
            }
            bounded_above_on_prefix(f, a, t + eta)
            bounded_prefix_contains(f, a, b, t + eta)
            bounded_prefix_set_contains_eq(f, a, b, t + eta)
            s.contains(t + eta)
            set_member_le_supremum(s, t, t + eta)
            t + eta <= t
            lt_add_pos(t, eta)
            t < t + eta
            lte_lt_trans(t + eta, t, t + eta)
            t + eta < t + eta
            not_lt_self(t + eta)
            false
        }
        not t < b
        not_lt_imp_gte[Real](t, b)
        t >= b
        b <= t
        lte_antisymm[Real](t, b)
        t = b
        // Part B: continuity at b gives a bound on the whole interval.
        lte_refl(b)
        closed_interval(a, b, b)
        closed_interval_set_contains_eq(a, b, b)
        closed_interval_set(a, b).contains(b)
        continuous_on_closed(f, a, b) = forall(x: Real) {
            closed_interval_set(a, b).contains(x) implies continuous_at(f, x)
        }
        closed_interval_set(a, b).contains(b) implies continuous_at(f, b)
        continuous_at(f, b)
        continuous_at_local_abs_bound(f, b)
        let (delta1: Real, bound1: Real) satisfy {
            delta1.is_positive and bound1.is_positive and forall(y: Real) {
                y.is_close(b, delta1) implies f(y).abs <= bound1
            }
        }
        neg_pos_is_neg(delta1)
        (-delta1).is_negative
        neg_lt_zero(-delta1)
        -delta1 < Real.0
        lt_add_right(-delta1, Real.0, b)
        -delta1 + b < Real.0 + b
        add_comm(-delta1, b)
        -delta1 + b = b + -delta1
        add_zero_left(b)
        Real.0 + b = b
        b + -delta1 < b
        b - delta1 = b + -delta1
        b - delta1 < b
        set_bound_below_supremum_not_upper(s, b, b - delta1)
        not is_set_upper_bound(s, b - delta1)
        set_not_upper_bound_witness(s, b - delta1)
        let sx: Real satisfy {
            s.contains(sx) and not sx <= b - delta1
        }
        s.contains(sx)
        not_lte_imp_gt[Real](sx, b - delta1)
        b - delta1 < sx
        set_member_le_supremum(s, b, sx)
        sx <= b
        s = bounded_prefix_set(f, a, b)
        bounded_prefix_set(f, a, b).contains(sx)
        bounded_prefix_member_in_interval(f, a, b, sx)
        a <= sx and sx <= b
        bounded_prefix_member_bound(f, a, b, sx)
        let m_s: Real satisfy {
            forall(y: Real) {
                closed_interval_set(a, sx).contains(y) implies f(y) <= m_s
            }
        }
        let bound = m_s.abs + bound1 + Real.1
        forall(y: Real) {
            if closed_interval_set(a, b).contains(y) {
                closed_interval_set_lower_le(a, b, y)
                a <= y
                closed_interval_set_le_upper(a, b, y)
                y <= b
                if y <= sx {
                    closed_interval(a, sx, y)
                    closed_interval_set_contains_eq(a, sx, y)
                    closed_interval_set(a, sx).contains(y)
                    forall(y0: Real) {
                        closed_interval_set(a, sx).contains(y0) implies f(y0) <= m_s
                    }
                    f(y) <= m_s
                    lte_abs(m_s)
                    m_s <= m_s.abs
                    abs_gte_zero(m_s)
                    m_s.abs >= Real.0
                    bound1.is_positive
                    pos_add_one_is_pos(bound1)
                    (bound1 + Real.1).is_positive
                    lt_add_pos(m_s.abs, bound1 + Real.1)
                    m_s.abs < m_s.abs + (bound1 + Real.1)
                    m_s.abs + (bound1 + Real.1) = m_s.abs + bound1 + Real.1
                    m_s.abs < m_s.abs + bound1 + Real.1
                    lt_imp_lte(m_s.abs, m_s.abs + bound1 + Real.1)
                    m_s.abs <= m_s.abs + bound1 + Real.1
                    lte_trans[Real](m_s, m_s.abs, m_s.abs + bound1 + Real.1)
                    m_s <= m_s.abs + bound1 + Real.1
                    lte_trans[Real](f(y), m_s, m_s.abs + bound1 + Real.1)
                    f(y) <= m_s.abs + bound1 + Real.1
                    f(y) <= bound
                } else {
                    not y <= sx
                    not_lte_imp_gt[Real](sx, y)
                    sx < y
                    lt_trans[Real](b - delta1, sx, y)
                    b - delta1 < y
                    lt_add_pos(b, delta1)
                    b < b + delta1
                    y <= b
                    lte_lt_trans(y, b, b + delta1)
                    y < b + delta1
                    bounds_imp_close(y, b, delta1)
                    y.is_close(b, delta1)
                    forall(y0: Real) {
                        y0.is_close(b, delta1) implies f(y0).abs <= bound1
                    }
                    y.is_close(b, delta1) implies f(y).abs <= bound1
                    f(y).abs <= bound1
                    lte_abs(f(y))
                    f(y) <= f(y).abs
                    lte_trans[Real](f(y), f(y).abs, bound1)
                    f(y) <= bound1
                    abs_add_one_is_pos(m_s)
                    (m_s.abs + Real.1).is_positive
                    lt_add_pos(bound1, m_s.abs + Real.1)
                    bound1 < bound1 + (m_s.abs + Real.1)
                    bound1 + (m_s.abs + Real.1) = bound1 + m_s.abs + Real.1
                    bound1 < bound1 + m_s.abs + Real.1
                    add_comm(bound1, m_s.abs)
                    bound1 + m_s.abs = m_s.abs + bound1
                    bound1 + m_s.abs + Real.1 = m_s.abs + bound1 + Real.1
                    bound1 < m_s.abs + bound1 + Real.1
                    lt_imp_lte(bound1, m_s.abs + bound1 + Real.1)
                    bound1 <= m_s.abs + bound1 + Real.1
                    lte_trans[Real](f(y), bound1, m_s.abs + bound1 + Real.1)
                    f(y) <= m_s.abs + bound1 + Real.1
                    f(y) <= bound
                }
            }
        }
        exists(bound3: Real) {
            forall(y: Real) {
                closed_interval_set(a, b).contains(y) implies f(y) <= bound3
            }
        }
    }
}

/// The pointwise reciprocal of the function x ↦ c - f(x).
define reciprocal_const_minus(c: Real, f: Real -> Real, x: Real) -> Real {
    (c - f(x)).inverse
}

/// The reciprocal of c - f is continuous wherever f is continuous and f < c.
theorem continuous_at_reciprocal_const_minus(c: Real, f: Real -> Real, x0: Real) {
    continuous_at(f, x0) and f(x0) < c implies
        continuous_at(reciprocal_const_minus(c, f), x0)
} by {
    define continuous_at_x(h: Real -> Real) -> Bool {
        continuous_at(h, x0)
    }
    if continuous_at(f, x0) and f(x0) < c {
        continuous_at_const_sub_left(c, f, x0)
        continuous_at(const_sub_left(c, f), x0)
        lt_imp_minus_pos(f(x0), c)
        (c - f(x0)).is_positive
        lt_imp_ne(c - f(x0), Real.0)
        const_sub_left(c, f, x0) = c - f(x0)
        const_sub_left(c, f, x0) != Real.0
        continuous_at_pointwise_reciprocal_real(const_sub_left(c, f), x0)
        continuous_at(pointwise_reciprocal_real(const_sub_left(c, f)), x0)
        continuous_at_x(pointwise_reciprocal_real(const_sub_left(c, f)))
        forall(x: Real) {
            reciprocal_const_minus(c, f, x) = (c - f(x)).inverse
            const_sub_left(c, f, x) = c - f(x)
            pointwise_reciprocal_real(const_sub_left(c, f), x) =
                const_sub_left(c, f, x).inverse
            reciprocal_const_minus(c, f, x) =
                pointwise_reciprocal_real(const_sub_left(c, f), x)
        }
        function_extensionality(reciprocal_const_minus(c, f),
            pointwise_reciprocal_real(const_sub_left(c, f)))
        reciprocal_const_minus(c, f) = pointwise_reciprocal_real(const_sub_left(c, f))
        function_eq_transport_predicate_rev(continuous_at_x,
            reciprocal_const_minus(c, f), pointwise_reciprocal_real(const_sub_left(c, f)))
        continuous_at(reciprocal_const_minus(c, f), x0)
    }
}

/// The inverse of the inverse of a nonzero real number is the number itself.
theorem inverse_inverse(x: Real) {
    x != Real.0 implies x.inverse.inverse = x
} by {
    if x != Real.0 {
        mul_inverse(x)
        x * x.inverse = Real.1
        if x.inverse = Real.0 {
            mul_zero_right(x)
            x * Real.0 = Real.0
            x * x.inverse = Real.0
            x * x.inverse = Real.1
            zero_is_different_than_one
            Real.0 != Real.1
            Real.0 = Real.1
            false
        }
        x.inverse != Real.0
        mul_inverse(x.inverse)
        x.inverse * x.inverse.inverse = Real.1
        real_mul_comm(x.inverse, x)
        x.inverse * x = x * x.inverse
        x.inverse * x = Real.1
        x.inverse * x.inverse.inverse = x.inverse * x
        mul_left_cancel(x.inverse, x.inverse.inverse, x)
        x.inverse.inverse = x
    }
}

/// A function continuous on a closed interval attains a maximum on it.
theorem continuous_closed_interval_attains_maximum(f: Real -> Real, a: Real, b: Real) {
    continuous_on_closed(f, a, b) and a < b
    implies exists(x0: Real) {
        closed_interval_set(a, b).contains(x0) and
        (forall(y: Real) { closed_interval_set(a, b).contains(y) implies f(y) <= f(x0) })
    }
} by {
    if continuous_on_closed(f, a, b) and a < b {
        lt_imp_lte(a, b)
        a <= b
        let s_img = function_image(f, closed_interval_set(a, b))
        // The image set is nonempty.
        closed_interval_set_contains_lower[Real](a, b)
        closed_interval_set(a, b).contains(a)
        maps_into_set_image(closed_interval_set(a, b), f, a)
        set_image(closed_interval_set(a, b), f).contains(f(a))
        function_image(f, closed_interval_set(a, b)) = set_image(closed_interval_set(a, b), f)
        s_img.contains(f(a))
        exists(x: Real) {
            s_img.contains(x)
        }
        is_nonempty(s_img)
        // The image set is bounded above.
        continuous_closed_interval_bounded_above(f, a, b)
        let bound: Real satisfy {
            forall(y: Real) {
                closed_interval_set(a, b).contains(y) implies f(y) <= bound
            }
        }
        forall(y: Real) {
            if s_img.contains(y) {
                set_image_contains_witness(closed_interval_set(a, b), f, y)
                let x: Real satisfy {
                    closed_interval_set(a, b).contains(x) and y = f(x)
                }
                closed_interval_set(a, b).contains(x)
                f(x) <= bound
                y = f(x)
                y <= bound
            }
        }
        function(s0: Set[Real], bound0: Real) {
            forall(x0: Real) {
                s0.contains(x0) implies x0 <= bound0
            } = is_set_upper_bound(s0, bound0)
        }(s_img, bound)
        is_set_upper_bound(s_img, bound)
        has_upper_bound(s_img)
        completeness(s_img)
        let m: Real satisfy {
            is_set_supremum(s_img, m)
        }
        is_set_supremum(s_img, m)
        // The supremum is attained: a non-attaining supremum contradicts boundedness
        // of the reciprocal of m - f.
        if forall(x: Real) { closed_interval_set(a, b).contains(x) implies f(x) < m } {
            forall(x0: Real) {
                if closed_interval_set(a, b).contains(x0) {
                    continuous_on_closed(f, a, b) = forall(x: Real) {
                        closed_interval_set(a, b).contains(x) implies continuous_at(f, x)
                    }
                    closed_interval_set(a, b).contains(x0) implies continuous_at(f, x0)
                    continuous_at(f, x0)
                    forall(x: Real) { closed_interval_set(a, b).contains(x) implies f(x) < m }
                    f(x0) < m
                    continuous_at_reciprocal_const_minus(m, f, x0)
                    continuous_at(reciprocal_const_minus(m, f), x0)
                }
            }
            continuous_on_closed(reciprocal_const_minus(m, f), a, b) = forall(x0: Real) {
                closed_interval_set(a, b).contains(x0) implies
                    continuous_at(reciprocal_const_minus(m, f), x0)
            }
            continuous_on_closed(reciprocal_const_minus(m, f), a, b)
            continuous_closed_interval_bounded_above(reciprocal_const_minus(m, f), a, b)
            let m_bound: Real satisfy {
                forall(y: Real) {
                    closed_interval_set(a, b).contains(y) implies
                        reciprocal_const_minus(m, f, y) <= m_bound
                }
            }
            // m_bound is positive.
            closed_interval_set_contains_lower[Real](a, b)
            closed_interval_set(a, b).contains(a)
            forall(x: Real) { closed_interval_set(a, b).contains(x) implies f(x) < m }
            f(a) < m
            reciprocal_const_minus(m, f, a) = (m - f(a)).inverse
            lt_imp_minus_pos(f(a), m)
            (m - f(a)).is_positive
            inverse_of_positive_is_positive[Real](m - f(a))
            Real.0 < (m - f(a)).inverse
            reciprocal_const_minus(m, f, a) > Real.0
            forall(y: Real) {
                closed_interval_set(a, b).contains(y) implies reciprocal_const_minus(m, f, y) <= m_bound
            }
            closed_interval_set(a, b).contains(a) implies reciprocal_const_minus(m, f, a) <= m_bound
            reciprocal_const_minus(m, f, a) <= m_bound
            lt_of_lt_of_lte[Real](Real.0, reciprocal_const_minus(m, f, a), m_bound)
            Real.0 < m_bound
            inverse_of_positive_is_positive[Real](m_bound)
            Real.0 < m_bound.inverse
            gt_zero_imp_pos(m_bound.inverse)
            m_bound.inverse.is_positive
            set_supremum_close_from_below(s_img, m, m_bound.inverse)
            let y: Real satisfy {
                s_img.contains(y) and m - m_bound.inverse < y and y <= m and
                y.is_close(m, m_bound.inverse)
            }
            s_img.contains(y)
            set_image_contains_witness(closed_interval_set(a, b), f, y)
            let x: Real satisfy {
                closed_interval_set(a, b).contains(x) and y = f(x)
            }
            closed_interval_set(a, b).contains(x)
            y = f(x)
            m - m_bound.inverse < y
            lt_add_right(m - m_bound.inverse, y, m_bound.inverse)
            (m - m_bound.inverse) + m_bound.inverse < y + m_bound.inverse
            (m - m_bound.inverse) + m_bound.inverse = m
            m < y + m_bound.inverse
            lt_add_right(m, y + m_bound.inverse, -f(x))
            m + -f(x) < (y + m_bound.inverse) + -f(x)
            m - f(x) = m + -f(x)
            y = f(x)
            add_comm(f(x), m_bound.inverse)
            f(x) + m_bound.inverse = m_bound.inverse + f(x)
            (f(x) + m_bound.inverse) + -f(x) = (m_bound.inverse + f(x)) + -f(x)
            sub_cancels(m_bound.inverse, f(x))
            (m_bound.inverse + f(x)) + -f(x) = m_bound.inverse
            (f(x) + m_bound.inverse) + -f(x) = m_bound.inverse
            (y + m_bound.inverse) + -f(x) = m_bound.inverse
            m - f(x) < m_bound.inverse
            forall(x0: Real) { closed_interval_set(a, b).contains(x0) implies f(x0) < m }
            f(x) < m
            lt_imp_minus_pos(f(x), m)
            (m - f(x)).is_positive
            pos_gt_zero(m - f(x))
            m - f(x) > Real.0
            real_inverse_antitone_pos_strict(m - f(x), m_bound.inverse)
            m_bound.inverse.inverse < (m - f(x)).inverse
            lt_imp_ne_symm(Real.0, m_bound)
            m_bound != Real.0
            inverse_inverse(m_bound)
            m_bound.inverse.inverse = m_bound
            m_bound < (m - f(x)).inverse
            reciprocal_const_minus(m, f, x) = (m - f(x)).inverse
            m_bound < reciprocal_const_minus(m, f, x)
            forall(z: Real) {
                closed_interval_set(a, b).contains(z) implies reciprocal_const_minus(m, f, z) <= m_bound
            }
            closed_interval_set(a, b).contains(x) implies reciprocal_const_minus(m, f, x) <= m_bound
            reciprocal_const_minus(m, f, x) <= m_bound
            false
        }
        // The supremum is attained at some point of the interval.
        not forall(x: Real) { closed_interval_set(a, b).contains(x) implies f(x) < m }
        let x0: Real satisfy {
            closed_interval_set(a, b).contains(x0) and not f(x0) < m
        }
        closed_interval_set(a, b).contains(x0)
        not_lt_imp_gte[Real](f(x0), m)
        f(x0) >= m
        m <= f(x0)
        maps_into_set_image(closed_interval_set(a, b), f, x0)
        set_image(closed_interval_set(a, b), f).contains(f(x0))
        function_image(f, closed_interval_set(a, b)) = set_image(closed_interval_set(a, b), f)
        s_img.contains(f(x0))
        set_member_le_supremum(s_img, m, f(x0))
        f(x0) <= m
        lte_antisymm[Real](f(x0), m)
        f(x0) = m
        forall(y: Real) {
            if closed_interval_set(a, b).contains(y) {
                maps_into_set_image(closed_interval_set(a, b), f, y)
                set_image(closed_interval_set(a, b), f).contains(f(y))
                function_image(f, closed_interval_set(a, b)) = set_image(closed_interval_set(a, b), f)
                s_img.contains(f(y))
                set_member_le_supremum(s_img, m, f(y))
                f(y) <= m
                m = f(x0)
                f(y) <= f(x0)
            }
        }
        exists(x1: Real) {
            closed_interval_set(a, b).contains(x1) and
            (forall(y: Real) { closed_interval_set(a, b).contains(y) implies f(y) <= f(x1) })
        }
    }
}

/// Negating both sides of a non-strict inequality reverses it.
theorem lte_imp_neg_lte_neg(a: Real, b: Real) {
    a <= b implies -b <= -a
} by {
    if a <= b {
        lte_add_right(a, b, -(a + b))
        a + -(a + b) <= b + -(a + b)
        neg_distrib(a, b)
        -(a + b) = -a + -b
        a + -(a + b) = a + (-a + -b)
        add_assoc(a, -a, -b)
        (a + -a) + -b = a + (-a + -b)
        add_neg_eq_zero(a)
        a + -a = Real.0
        add_zero_left(-b)
        Real.0 + -b = -b
        (a + -a) + -b = -b
        a + (-a + -b) = -b
        a + -(a + b) = -b
        neg_distrib(b, a)
        -(a + b) = -a + -b
        b + -(a + b) = b + (-a + -b)
        add_assoc(b, -a, -b)
        (b + -a) + -b = b + (-a + -b)
        add_comm(b, -a)
        b + -a = -a + b
        sub_cancels(-a, b)
        (-a + b) + -b = -a
        (b + -a) + -b = -a
        b + (-a + -b) = -a
        b + -(a + b) = -a
        -b <= -a
    }
}

/// A function continuous on a closed interval attains a minimum on it.
theorem continuous_closed_interval_attains_minimum(f: Real -> Real, a: Real, b: Real) {
    continuous_on_closed(f, a, b) and a < b
    implies exists(x0: Real) {
        closed_interval_set(a, b).contains(x0) and
        (forall(y: Real) { closed_interval_set(a, b).contains(y) implies f(x0) <= f(y) })
    }
} by {
    if continuous_on_closed(f, a, b) and a < b {
        forall(x: Real) {
            if closed_interval_set(a, b).contains(x) {
                continuous_on_closed(f, a, b) = forall(x0: Real) {
                    closed_interval_set(a, b).contains(x0) implies continuous_at(f, x0)
                }
                closed_interval_set(a, b).contains(x) implies continuous_at(f, x)
                continuous_at(f, x)
                continuous_at_pointwise_neg(f, x)
                continuous_at(pointwise_neg(f), x)
            }
        }
        continuous_on_closed(pointwise_neg(f), a, b) = forall(x: Real) {
            closed_interval_set(a, b).contains(x) implies continuous_at(pointwise_neg(f), x)
        }
        continuous_on_closed(pointwise_neg(f), a, b)
        continuous_closed_interval_attains_maximum(pointwise_neg(f), a, b)
        let x0: Real satisfy {
            closed_interval_set(a, b).contains(x0) and
            (forall(y: Real) { closed_interval_set(a, b).contains(y) implies
                pointwise_neg(f, y) <= pointwise_neg(f, x0) })
        }
        closed_interval_set(a, b).contains(x0)
        forall(y: Real) {
            if closed_interval_set(a, b).contains(y) {
                forall(y0: Real) { closed_interval_set(a, b).contains(y0) implies pointwise_neg(f, y0) <= pointwise_neg(f, x0) }
                pointwise_neg(f, y) <= pointwise_neg(f, x0)
                pointwise_neg(f, y) = -f(y)
                pointwise_neg(f, x0) = -f(x0)
                -f(y) <= -f(x0)
                lte_imp_neg_lte_neg(-f(y), -f(x0))
                -f(y) <= -f(x0) implies -(-f(x0)) <= -(-f(y))
                -(-f(x0)) <= -(-f(y))
                neg_neg(f(x0))
                -(-f(x0)) = f(x0)
                neg_neg(f(y))
                -(-f(y)) = f(y)
                f(x0) <= f(y)
            }
        }
        exists(x1: Real) {
            closed_interval_set(a, b).contains(x1) and
            (forall(y: Real) { closed_interval_set(a, b).contains(y) implies f(x1) <= f(y) })
        }
    }
}

/// A function constant on a closed interval has derivative zero at every interior point.
theorem constant_on_interval_has_derivative_zero(f: Real -> Real, a: Real, b: Real, c: Real) {
    a < c and c < b and
    (forall(y: Real) { closed_interval_set(a, b).contains(y) implies f(y) = f(c) })
    implies has_derivative_at(f, c, Real.0)
} by {
    if a < c and c < b and
       (forall(y: Real) { closed_interval_set(a, b).contains(y) implies f(y) = f(c) }) {
        has_derivative_at(f, c, Real.0) = forall(eps: Real) {
            eps.is_positive implies exists(delta: Real) {
                delta.is_positive and forall(x: Real) {
                    x != c and x.is_close(c, delta)
                    implies difference_quotient(f, c, x).is_close(Real.0, eps)
                }
            }
        }
        forall(eps: Real) {
            if eps.is_positive {
                lt_imp_minus_pos(a, c)
                (c - a).is_positive
                lt_imp_minus_pos(c, b)
                (b - c).is_positive
                eps_smaller_than_both(c - a, b - c)
                let delta: Real satisfy {
                    delta.is_positive and delta < c - a and delta < b - c
                }
                delta.is_positive
                forall(x: Real) {
                    if x != c and x.is_close(c, delta) {
                        close_imp_bounds(x, c, delta)
                        c - delta < x and x < c + delta
                        lt_add_right(delta, c - a, a)
                        delta + a < (c - a) + a
                        (c - a) + a = c
                        delta + a < c
                        add_comm(delta, a)
                        delta + a = a + delta
                        a + delta < c
                        (c - delta) + delta = c
                        a + delta < (c - delta) + delta
                        lt_add_converse(a, c - delta, delta)
                        a < c - delta
                        lt_trans[Real](a, c - delta, x)
                        a < x
                        lt_add_right(delta, b - c, c)
                        delta + c < (b - c) + c
                        (b - c) + c = b
                        delta + c < b
                        add_comm(delta, c)
                        delta + c = c + delta
                        c + delta < b
                        lt_trans[Real](x, c + delta, b)
                        x < b
                        lt_imp_lte(a, x)
                        a <= x
                        lt_imp_lte(x, b)
                        x <= b
                        closed_interval(a, b, x)
                        closed_interval_set_contains_eq(a, b, x)
                        closed_interval_set(a, b).contains(x)
                        forall(y: Real) { closed_interval_set(a, b).contains(y) implies f(y) = f(c) }
                        closed_interval_set(a, b).contains(x) implies f(x) = f(c)
                        f(x) = f(c)
                        f(x) - f(c) = Real.0
                        sub_ne_zero_of_ne(x, c)
                        x - c != Real.0
                        difference_quotient(f, c, x) = (f(x) - f(c)) / (x - c)
                        (f(x) - f(c)) / (x - c) = (f(x) - f(c)) * (x - c).inverse
                        mul_zero_left((x - c).inverse)
                        Real.0 * (x - c).inverse = Real.0
                        (f(x) - f(c)) * (x - c).inverse = Real.0
                        difference_quotient(f, c, x) = Real.0
                        self_close(Real.0, eps)
                        Real.0.is_close(Real.0, eps)
                        difference_quotient(f, c, x).is_close(Real.0, eps)
                    }
                }
                exists(delta2: Real) {
                    delta2.is_positive and forall(x: Real) {
                        x != c and x.is_close(c, delta2)
                        implies difference_quotient(f, c, x).is_close(Real.0, eps)
                    }
                }
            }
        }
        forall(eps: Real) {
            eps.is_positive implies exists(delta: Real) {
                delta.is_positive and forall(x: Real) {
                    x != c and x.is_close(c, delta)
                    implies difference_quotient(f, c, x).is_close(Real.0, eps)
                }
            }
        }
        if not has_derivative_at(f, c, Real.0) {
            has_derivative_at(f, c, Real.0) = forall(eps: Real) {
                eps.is_positive implies exists(delta: Real) {
                    delta.is_positive and forall(x: Real) {
                        x != c and x.is_close(c, delta)
                        implies difference_quotient(f, c, x).is_close(Real.0, eps)
                    }
                }
            }
            not forall(eps: Real) {
                eps.is_positive implies exists(delta: Real) {
                    delta.is_positive and forall(x: Real) {
                        x != c and x.is_close(c, delta)
                        implies difference_quotient(f, c, x).is_close(Real.0, eps)
                    }
                }
            }
            let bad_eps: Real satisfy {
                bad_eps.is_positive and forall(delta: Real) {
                    not (delta.is_positive and forall(x: Real) {
                        x != c and x.is_close(c, delta)
                        implies difference_quotient(f, c, x).is_close(Real.0, bad_eps)
                    })
                }
            }
            forall(eps: Real) {
                eps.is_positive implies exists(delta: Real) {
                    delta.is_positive and forall(x: Real) {
                        x != c and x.is_close(c, delta)
                        implies difference_quotient(f, c, x).is_close(Real.0, eps)
                    }
                }
            }
            bad_eps.is_positive implies exists(delta: Real) {
                delta.is_positive and forall(x: Real) {
                    x != c and x.is_close(c, delta)
                    implies difference_quotient(f, c, x).is_close(Real.0, bad_eps)
                }
            }
            exists(delta: Real) {
                delta.is_positive and forall(x: Real) {
                    x != c and x.is_close(c, delta)
                    implies difference_quotient(f, c, x).is_close(Real.0, bad_eps)
                }
            }
            false
        }
        has_derivative_at(f, c, Real.0)
    }
}

/// Rolle's theorem: a function differentiable on (a, b) and continuous on [a, b]
/// with equal endpoint values has a zero derivative somewhere inside.
theorem rolle_theorem(f: Real -> Real, a: Real, b: Real) {
    continuous_on_closed(f, a, b) and a < b and differentiable_on_open(f, a, b) and
    f(a) = f(b)
    implies exists(c: Real) {
        a < c and c < b and has_derivative_at(f, c, Real.0)
    }
} by {
    if continuous_on_closed(f, a, b) and a < b and differentiable_on_open(f, a, b) and
       f(a) = f(b) {
        continuous_closed_interval_attains_maximum(f, a, b)
        let x_max: Real satisfy {
            closed_interval_set(a, b).contains(x_max) and
            (forall(y: Real) { closed_interval_set(a, b).contains(y) implies f(y) <= f(x_max) })
        }
        closed_interval_set(a, b).contains(x_max)
        closed_interval_set_lower_le(a, b, x_max)
        a <= x_max
        closed_interval_set_le_upper(a, b, x_max)
        x_max <= b
        continuous_closed_interval_attains_minimum(f, a, b)
        let x_min: Real satisfy {
            closed_interval_set(a, b).contains(x_min) and
            (forall(y: Real) { closed_interval_set(a, b).contains(y) implies f(x_min) <= f(y) })
        }
        closed_interval_set(a, b).contains(x_min)
        closed_interval_set_lower_le(a, b, x_min)
        a <= x_min
        closed_interval_set_le_upper(a, b, x_min)
        x_min <= b
        if a < x_max and x_max < b {
            differentiable_on_open(f, a, b) = forall(x: Real) {
                a < x and x < b implies differentiable_at(f, x)
            }
            a < x_max and x_max < b implies differentiable_at(f, x_max)
            differentiable_at(f, x_max)
            differentiable_at(f, x_max) = exists(d: Real) {
                has_derivative_at(f, x_max, d)
            }
            let d: Real satisfy {
                has_derivative_at(f, x_max, d)
            }
            forall(y: Real) {
                if a < y and y < b {
                    lt_imp_lte(a, y)
                    a <= y
                    lt_imp_lte(y, b)
                    y <= b
                    closed_interval(a, b, y)
                    closed_interval_set_contains_eq(a, b, y)
                    closed_interval_set(a, b).contains(y)
                    forall(y0: Real) { closed_interval_set(a, b).contains(y0) implies f(y0) <= f(x_max) }
                    f(y) <= f(x_max)
                }
            }
            fermat_interior_maximum(f, a, b, x_max, d)
            d = Real.0
            has_derivative_at(f, x_max, d)
            has_derivative_at(f, x_max, Real.0)
            exists(c: Real) {
                a < c and c < b and has_derivative_at(f, c, Real.0)
            }
        } else {
            if a < x_min and x_min < b {
            differentiable_on_open(f, a, b) = forall(x: Real) {
                a < x and x < b implies differentiable_at(f, x)
            }
            a < x_min and x_min < b implies differentiable_at(f, x_min)
            differentiable_at(f, x_min)
            differentiable_at(f, x_min) = exists(d: Real) {
                has_derivative_at(f, x_min, d)
            }
            let d: Real satisfy {
                has_derivative_at(f, x_min, d)
            }
            derivative_pointwise_neg(f, x_min, d)
            has_derivative_at(pointwise_neg(f), x_min, -d)
            forall(y: Real) {
                if a < y and y < b {
                    lt_imp_lte(a, y)
                    a <= y
                    lt_imp_lte(y, b)
                    y <= b
                    closed_interval(a, b, y)
                    closed_interval_set_contains_eq(a, b, y)
                    closed_interval_set(a, b).contains(y)
                    forall(y0: Real) { closed_interval_set(a, b).contains(y0) implies f(x_min) <= f(y0) }
                    f(x_min) <= f(y)
                    lte_imp_neg_lte_neg(f(x_min), f(y))
                    -f(y) <= -f(x_min)
                    pointwise_neg(f, y) = -f(y)
                    pointwise_neg(f, x_min) = -f(x_min)
                    pointwise_neg(f, y) <= pointwise_neg(f, x_min)
                }
            }
            fermat_interior_maximum(pointwise_neg(f), a, b, x_min, -d)
            -d = Real.0
            neg_zero
            -Real.0 = Real.0
            -d = -Real.0
            neg_neg(d)
            -(-d) = d
            neg_neg(Real.0)
            -(-Real.0) = Real.0
            -(-d) = -(-Real.0)
            d = Real.0
            has_derivative_at(f, x_min, d)
            has_derivative_at(f, x_min, Real.0)
            exists(c: Real) {
                a < c and c < b and has_derivative_at(f, c, Real.0)
            }
        } else {
            // Both extrema are attained at endpoints, so f is constant on [a, b].
            not (a < x_max and x_max < b)
            not (a < x_min and x_min < b)
            if a < x_max {
                not (a < x_max and x_max < b)
                not x_max < b
                not_lt_imp_gte[Real](x_max, b)
                x_max >= b
                b <= x_max
                lte_antisymm[Real](x_max, b)
                x_max = b
                f(x_max) = f(b)
                f(b) = f(a)
                f(x_max) = f(a)
            } else {
                not a < x_max
                not_lt_imp_gte[Real](a, x_max)
                x_max <= a
                lte_antisymm[Real](a, x_max)
                a = x_max
                x_max = a
                f(x_max) = f(a)
            }
            f(x_max) = f(a)
            if a < x_min {
                not (a < x_min and x_min < b)
                not x_min < b
                not_lt_imp_gte[Real](x_min, b)
                x_min >= b
                b <= x_min
                lte_antisymm[Real](x_min, b)
                x_min = b
                f(x_min) = f(b)
                f(b) = f(a)
                f(x_min) = f(a)
            } else {
                not a < x_min
                not_lt_imp_gte[Real](a, x_min)
                x_min <= a
                lte_antisymm[Real](a, x_min)
                a = x_min
                x_min = a
                f(x_min) = f(a)
            }
            f(x_min) = f(a)
            forall(y: Real) {
                if closed_interval_set(a, b).contains(y) {
                    forall(y0: Real) { closed_interval_set(a, b).contains(y0) implies f(x_min) <= f(y0) }
                    f(x_min) <= f(y)
                    forall(y0: Real) { closed_interval_set(a, b).contains(y0) implies f(y0) <= f(x_max) }
                    f(y) <= f(x_max)
                    f(x_max) = f(a)
                    f(x_min) = f(a)
                    f(x_max) = f(x_min)
                    f(y) <= f(x_min)
                    lte_antisymm[Real](f(y), f(x_min))
                    f(y) = f(x_min)
                    f(x_min) = f(a)
                    f(y) = f(a)
                }
            }
            add_real_eps_between(a, b)
            let eps0: Real satisfy {
                eps0.is_positive and a + eps0 < b
            }
            let c = a + eps0
            lt_add_pos(a, eps0)
            a < c
            c < b
            a < c and c < b
            lt_imp_lte(a, c)
            a <= c
            lt_imp_lte(c, b)
            c <= b
            closed_interval(a, b, c)
            closed_interval_set_contains_eq(a, b, c)
            closed_interval_set(a, b).contains(c)
            forall(y0: Real) { closed_interval_set(a, b).contains(y0) implies f(y0) = f(a) }
            closed_interval_set(a, b).contains(c) implies f(c) = f(a)
            f(c) = f(a)
            forall(y: Real) {
                if closed_interval_set(a, b).contains(y) {
                    forall(y0: Real) { closed_interval_set(a, b).contains(y0) implies f(y0) = f(a) }
                    f(y) = f(a)
                    f(a) = f(c)
                    f(y) = f(c)
                }
            }
            constant_on_interval_has_derivative_zero(f, a, b, c)
            has_derivative_at(f, c, Real.0)
            exists(c2: Real) {
                a < c2 and c2 < b and has_derivative_at(f, c2, Real.0)
            }
            }
        }
    }
}

/// The slope of the chord through (a, f(a)) and (b, f(b)).
define secant_slope(f: Real -> Real, a: Real, b: Real) -> Real {
    (f(b) - f(a)) / (b - a)
}

/// The function f minus the chord through (a, f(a)) and (b, f(b)).
define secant_remainder(f: Real -> Real, a: Real, b: Real, x: Real) -> Real {
    f(x) - secant_slope(f, a, b) * (x - a)
}

/// The pointwise derivative of the secant remainder: df minus the chord slope.
define secant_remainder_derivative(df: Real -> Real, f: Real -> Real, a: Real, b: Real, x: Real) -> Real {
    df(x) - secant_slope(f, a, b)
}

/// The secant remainder is the pointwise difference of f and an affine function.
theorem secant_remainder_eq_pointwise(f: Real -> Real, a: Real, b: Real) {
    secant_remainder(f, a, b) = pointwise_add(f, pointwise_neg(
        affine_real(secant_slope(f, a, b), -(secant_slope(f, a, b)) * a)))
} by {
    forall(x: Real) {
        secant_remainder(f, a, b, x) = f(x) - secant_slope(f, a, b) * (x - a)
        x - a = x + -a
        secant_slope(f, a, b) * (x - a) = secant_slope(f, a, b) * (x + -a)
        mul_distrib_right(secant_slope(f, a, b), x, -a)
        secant_slope(f, a, b) * (x + -a) =
            secant_slope(f, a, b) * x + secant_slope(f, a, b) * -a
        mul_neg_right(secant_slope(f, a, b), a)
        secant_slope(f, a, b) * -a = -(secant_slope(f, a, b) * a)
        mul_neg_left(secant_slope(f, a, b), a)
        -(secant_slope(f, a, b)) * a = -(secant_slope(f, a, b) * a)
        affine_real(secant_slope(f, a, b), -(secant_slope(f, a, b)) * a, x) =
            secant_slope(f, a, b) * x + -(secant_slope(f, a, b)) * a
        affine_real(secant_slope(f, a, b), -(secant_slope(f, a, b)) * a, x) =
            secant_slope(f, a, b) * x + -(secant_slope(f, a, b) * a)
        secant_slope(f, a, b) * x + secant_slope(f, a, b) * -a =
            affine_real(secant_slope(f, a, b), -(secant_slope(f, a, b)) * a, x)
        secant_slope(f, a, b) * (x + -a) =
            affine_real(secant_slope(f, a, b), -(secant_slope(f, a, b)) * a, x)
        secant_slope(f, a, b) * (x - a) =
            affine_real(secant_slope(f, a, b), -(secant_slope(f, a, b)) * a, x)
        secant_remainder(f, a, b, x) =
            f(x) - affine_real(secant_slope(f, a, b), -(secant_slope(f, a, b)) * a, x)
        f(x) - affine_real(secant_slope(f, a, b), -(secant_slope(f, a, b)) * a, x) =
            f(x) + -(affine_real(secant_slope(f, a, b), -(secant_slope(f, a, b)) * a, x))
        pointwise_neg(affine_real(secant_slope(f, a, b), -(secant_slope(f, a, b)) * a), x) =
            -(affine_real(secant_slope(f, a, b), -(secant_slope(f, a, b)) * a, x))
        pointwise_add(f, pointwise_neg(affine_real(secant_slope(f, a, b), -(secant_slope(f, a, b)) * a)), x) =
            f(x) + pointwise_neg(affine_real(secant_slope(f, a, b), -(secant_slope(f, a, b)) * a), x)
        pointwise_add(f, pointwise_neg(affine_real(secant_slope(f, a, b), -(secant_slope(f, a, b)) * a)), x) =
            f(x) + -(affine_real(secant_slope(f, a, b), -(secant_slope(f, a, b)) * a, x))
        secant_remainder(f, a, b, x) =
            pointwise_add(f, pointwise_neg(affine_real(secant_slope(f, a, b), -(secant_slope(f, a, b)) * a)), x)
    }
    function_extensionality(secant_remainder(f, a, b),
        pointwise_add(f, pointwise_neg(
            affine_real(secant_slope(f, a, b), -(secant_slope(f, a, b)) * a))))
    secant_remainder(f, a, b) = pointwise_add(f, pointwise_neg(
        affine_real(secant_slope(f, a, b), -(secant_slope(f, a, b)) * a)))
}

/// The secant remainder is continuous on the closed interval.
theorem secant_remainder_continuous_on_closed(f: Real -> Real, a: Real, b: Real) {
    continuous(f) implies continuous_on_closed(secant_remainder(f, a, b), a, b)
} by {
    if continuous(f) {
        forall(x: Real) {
            if closed_interval_set(a, b).contains(x) {
                continuous_imp_continuous_at(f, x)
                continuous_at(f, x)
                continuous_at_affine_real(secant_slope(f, a, b), -(secant_slope(f, a, b)) * a, x)
                continuous_at(affine_real(secant_slope(f, a, b), -(secant_slope(f, a, b)) * a), x)
                continuous_at_pointwise_neg(affine_real(secant_slope(f, a, b), -(secant_slope(f, a, b)) * a), x)
                continuous_at(pointwise_neg(affine_real(secant_slope(f, a, b), -(secant_slope(f, a, b)) * a)), x)
                continuous_at_pointwise_add(f,
                    pointwise_neg(affine_real(secant_slope(f, a, b), -(secant_slope(f, a, b)) * a)), x)
                continuous_at(pointwise_add(f,
                    pointwise_neg(affine_real(secant_slope(f, a, b), -(secant_slope(f, a, b)) * a))), x)
                secant_remainder_eq_pointwise(f, a, b)
                secant_remainder(f, a, b) = pointwise_add(f, pointwise_neg(
                    affine_real(secant_slope(f, a, b), -(secant_slope(f, a, b)) * a)))
                function_eq_transport_predicate_rev(
                    function(h: Real -> Real) { continuous_at(h, x) },
                    secant_remainder(f, a, b),
                    pointwise_add(f, pointwise_neg(
                        affine_real(secant_slope(f, a, b), -(secant_slope(f, a, b)) * a))))
                continuous_at(secant_remainder(f, a, b), x)
            }
        }
        continuous_on_closed(secant_remainder(f, a, b), a, b) = forall(x: Real) {
            closed_interval_set(a, b).contains(x) implies continuous_at(secant_remainder(f, a, b), x)
        }
        continuous_on_closed(secant_remainder(f, a, b), a, b)
    }
}

/// The secant remainder is differentiable on the open interval with the expected derivative.
theorem secant_remainder_is_derivative_on_open(f: Real -> Real, df: Real -> Real, a: Real, b: Real) {
    is_derivative_fn(f, df)
    implies is_derivative_on_open(secant_remainder(f, a, b),
        secant_remainder_derivative(df, f, a, b), a, b)
} by {
    if is_derivative_fn(f, df) {
        forall(x: Real) {
            if a < x and x < b {
                is_derivative_fn_at(f, df, x)
                has_derivative_at(f, x, df(x))
                affine_real_has_derivative_at(secant_slope(f, a, b), -(secant_slope(f, a, b)) * a, x)
                has_derivative_at(affine_real(secant_slope(f, a, b), -(secant_slope(f, a, b)) * a), x,
                    secant_slope(f, a, b))
                derivative_pointwise_sub(f,
                    affine_real(secant_slope(f, a, b), -(secant_slope(f, a, b)) * a),
                    x, df(x), secant_slope(f, a, b))
                has_derivative_at(pointwise_add(f,
                    pointwise_neg(affine_real(secant_slope(f, a, b), -(secant_slope(f, a, b)) * a))),
                    x, df(x) + -secant_slope(f, a, b))
                df(x) + -secant_slope(f, a, b) = df(x) - secant_slope(f, a, b)
                has_derivative_at(pointwise_add(f,
                    pointwise_neg(affine_real(secant_slope(f, a, b), -(secant_slope(f, a, b)) * a))),
                    x, df(x) - secant_slope(f, a, b))
                secant_remainder_eq_pointwise(f, a, b)
                secant_remainder(f, a, b) = pointwise_add(f, pointwise_neg(
                    affine_real(secant_slope(f, a, b), -(secant_slope(f, a, b)) * a)))
                secant_remainder_derivative(df, f, a, b, x) = df(x) - secant_slope(f, a, b)
                function_eq_transport_predicate_rev(
                    function(h: Real -> Real) {
                        has_derivative_at(h, x, secant_remainder_derivative(df, f, a, b, x))
                    },
                    secant_remainder(f, a, b),
                    pointwise_add(f, pointwise_neg(
                        affine_real(secant_slope(f, a, b), -(secant_slope(f, a, b)) * a))))
                has_derivative_at(secant_remainder(f, a, b), x,
                    secant_remainder_derivative(df, f, a, b, x))
            }
        }
        is_derivative_on_open(secant_remainder(f, a, b),
            secant_remainder_derivative(df, f, a, b), a, b) = forall(x: Real) {
            a < x and x < b implies has_derivative_at(secant_remainder(f, a, b), x,
                secant_remainder_derivative(df, f, a, b, x))
        }
        is_derivative_on_open(secant_remainder(f, a, b),
            secant_remainder_derivative(df, f, a, b), a, b)
    }
}

/// The secant remainder is continuous on the closed interval, given only
/// continuity of f on that closed interval.
theorem secant_remainder_continuous_on_closed_local(f: Real -> Real, a: Real, b: Real) {
    continuous_on_closed(f, a, b)
    implies continuous_on_closed(secant_remainder(f, a, b), a, b)
} by {
    if continuous_on_closed(f, a, b) {
        forall(x: Real) {
            if closed_interval_set(a, b).contains(x) {
                continuous_on_closed(f, a, b) = forall(y: Real) {
                    closed_interval_set(a, b).contains(y) implies continuous_at(f, y)
                }
                closed_interval_set(a, b).contains(x) implies continuous_at(f, x)
                continuous_at(f, x)
                continuous_at_affine_real(secant_slope(f, a, b), -(secant_slope(f, a, b)) * a, x)
                continuous_at(affine_real(secant_slope(f, a, b), -(secant_slope(f, a, b)) * a), x)
                continuous_at_pointwise_neg(affine_real(secant_slope(f, a, b), -(secant_slope(f, a, b)) * a), x)
                continuous_at(pointwise_neg(affine_real(secant_slope(f, a, b), -(secant_slope(f, a, b)) * a)), x)
                continuous_at_pointwise_add(f,
                    pointwise_neg(affine_real(secant_slope(f, a, b), -(secant_slope(f, a, b)) * a)), x)
                continuous_at(pointwise_add(f,
                    pointwise_neg(affine_real(secant_slope(f, a, b), -(secant_slope(f, a, b)) * a))), x)
                secant_remainder_eq_pointwise(f, a, b)
                secant_remainder(f, a, b) = pointwise_add(f, pointwise_neg(
                    affine_real(secant_slope(f, a, b), -(secant_slope(f, a, b)) * a)))
                function_eq_transport_predicate_rev(
                    function(h: Real -> Real) { continuous_at(h, x) },
                    secant_remainder(f, a, b),
                    pointwise_add(f, pointwise_neg(
                        affine_real(secant_slope(f, a, b), -(secant_slope(f, a, b)) * a))))
                continuous_at(secant_remainder(f, a, b), x)
            }
        }
        continuous_on_closed(secant_remainder(f, a, b), a, b) = forall(x: Real) {
            closed_interval_set(a, b).contains(x) implies continuous_at(secant_remainder(f, a, b), x)
        }
        continuous_on_closed(secant_remainder(f, a, b), a, b)
    }
}

/// The secant remainder is differentiable on the open interval with the
/// expected derivative, given only a pointwise derivative on that open
/// interval.
theorem secant_remainder_is_derivative_on_open_local(f: Real -> Real, df: Real -> Real, a: Real, b: Real) {
    is_derivative_on_open(f, df, a, b)
    implies is_derivative_on_open(secant_remainder(f, a, b),
        secant_remainder_derivative(df, f, a, b), a, b)
} by {
    if is_derivative_on_open(f, df, a, b) {
        forall(x: Real) {
            if a < x and x < b {
                is_derivative_on_open(f, df, a, b) = forall(y: Real) {
                    a < y and y < b implies has_derivative_at(f, y, df(y))
                }
                a < x and x < b implies has_derivative_at(f, x, df(x))
                has_derivative_at(f, x, df(x))
                affine_real_has_derivative_at(secant_slope(f, a, b), -(secant_slope(f, a, b)) * a, x)
                has_derivative_at(affine_real(secant_slope(f, a, b), -(secant_slope(f, a, b)) * a), x,
                    secant_slope(f, a, b))
                derivative_pointwise_sub(f,
                    affine_real(secant_slope(f, a, b), -(secant_slope(f, a, b)) * a),
                    x, df(x), secant_slope(f, a, b))
                has_derivative_at(pointwise_add(f,
                    pointwise_neg(affine_real(secant_slope(f, a, b), -(secant_slope(f, a, b)) * a))),
                    x, df(x) + -secant_slope(f, a, b))
                df(x) + -secant_slope(f, a, b) = df(x) - secant_slope(f, a, b)
                has_derivative_at(pointwise_add(f,
                    pointwise_neg(affine_real(secant_slope(f, a, b), -(secant_slope(f, a, b)) * a))),
                    x, df(x) - secant_slope(f, a, b))
                secant_remainder_eq_pointwise(f, a, b)
                secant_remainder(f, a, b) = pointwise_add(f, pointwise_neg(
                    affine_real(secant_slope(f, a, b), -(secant_slope(f, a, b)) * a)))
                secant_remainder_derivative(df, f, a, b, x) = df(x) - secant_slope(f, a, b)
                function_eq_transport_predicate_rev(
                    function(h: Real -> Real) {
                        has_derivative_at(h, x, secant_remainder_derivative(df, f, a, b, x))
                    },
                    secant_remainder(f, a, b),
                    pointwise_add(f, pointwise_neg(
                        affine_real(secant_slope(f, a, b), -(secant_slope(f, a, b)) * a))))
                has_derivative_at(secant_remainder(f, a, b), x,
                    secant_remainder_derivative(df, f, a, b, x))
            }
        }
        is_derivative_on_open(secant_remainder(f, a, b),
            secant_remainder_derivative(df, f, a, b), a, b) = forall(x: Real) {
            a < x and x < b implies has_derivative_at(secant_remainder(f, a, b), x,
                secant_remainder_derivative(df, f, a, b, x))
        }
        is_derivative_on_open(secant_remainder(f, a, b),
            secant_remainder_derivative(df, f, a, b), a, b)
    }
}

/// A pointwise derivative on an open interval makes the function differentiable there.
theorem is_derivative_on_open_imp_differentiable_on_open(
    f: Real -> Real, df: Real -> Real, a: Real, b: Real
) {
    is_derivative_on_open(f, df, a, b) implies differentiable_on_open(f, a, b)
} by {
    if is_derivative_on_open(f, df, a, b) {
        forall(x: Real) {
            if a < x and x < b {
                is_derivative_on_open(f, df, a, b) = forall(y: Real) {
                    a < y and y < b implies has_derivative_at(f, y, df(y))
                }
                a < x and x < b implies has_derivative_at(f, x, df(x))
                has_derivative_at(f, x, df(x))
                differentiable_at(f, x) = exists(d: Real) { has_derivative_at(f, x, d) }
                exists(d: Real) { has_derivative_at(f, x, d) }
                differentiable_at(f, x)
            }
        }
        differentiable_on_open(f, a, b) = forall(x: Real) {
            a < x and x < b implies differentiable_at(f, x)
        }
        differentiable_on_open(f, a, b)
    }
}

/// The secant remainder takes equal values at the endpoints.
theorem secant_remainder_endpoints_equal(f: Real -> Real, a: Real, b: Real) {
    a < b implies secant_remainder(f, a, b, a) = secant_remainder(f, a, b, b)
} by {
    if a < b {
        secant_remainder(f, a, b, a) = f(a) - secant_slope(f, a, b) * (a - a)
        a - a = Real.0
        secant_slope(f, a, b) * (a - a) = secant_slope(f, a, b) * Real.0
        mul_zero_right(secant_slope(f, a, b))
        secant_slope(f, a, b) * Real.0 = Real.0
        f(a) - secant_slope(f, a, b) * (a - a) = f(a) - Real.0
        f(a) - Real.0 = f(a) + -Real.0
        neg_zero
        -Real.0 = Real.0
        f(a) + -Real.0 = f(a) + Real.0
        add_zero_right(f(a))
        f(a) + Real.0 = f(a)
        f(a) - Real.0 = f(a)
        secant_remainder(f, a, b, a) = f(a)
        secant_remainder(f, a, b, b) = f(b) - secant_slope(f, a, b) * (b - a)
        lt_imp_ne_symm(a, b)
        b != a
        sub_ne_zero_of_ne(b, a)
        b - a != Real.0
        div_mul_cancel_denominator(f(b) - f(a), b - a)
        ((f(b) - f(a)) / (b - a)) * (b - a) = f(b) - f(a)
        secant_slope(f, a, b) = (f(b) - f(a)) / (b - a)
        secant_slope(f, a, b) * (b - a) = ((f(b) - f(a)) / (b - a)) * (b - a)
        secant_slope(f, a, b) * (b - a) = f(b) - f(a)
        f(b) - secant_slope(f, a, b) * (b - a) = f(b) - (f(b) - f(a))
        f(b) - (f(b) - f(a)) = f(b) + -(f(b) - f(a))
        f(b) - f(a) = f(b) + -f(a)
        -(f(b) - f(a)) = -(f(b) + -f(a))
        neg_distrib(f(b), -f(a))
        -(f(b) + -f(a)) = -f(b) + -(-f(a))
        neg_neg(f(a))
        -(-f(a)) = f(a)
        -f(b) + -(-f(a)) = -f(b) + f(a)
        -(f(b) - f(a)) = -f(b) + f(a)
        f(b) + -(f(b) - f(a)) = f(b) + (-f(b) + f(a))
        add_assoc(f(b), -f(b), f(a))
        (f(b) + -f(b)) + f(a) = f(b) + (-f(b) + f(a))
        add_neg_eq_zero(f(b))
        f(b) + -f(b) = Real.0
        add_zero_left(f(a))
        Real.0 + f(a) = f(a)
        (f(b) + -f(b)) + f(a) = f(a)
        f(b) + (-f(b) + f(a)) = f(a)
        f(b) + -(f(b) - f(a)) = f(a)
        f(b) - (f(b) - f(a)) = f(a)
        f(b) - secant_slope(f, a, b) * (b - a) = f(a)
        secant_remainder(f, a, b, b) = f(a)
        secant_remainder(f, a, b, a) = secant_remainder(f, a, b, b)
    }
}

/// The mean value theorem: a function continuous everywhere and differentiable
/// everywhere has a point where its derivative equals the chord slope.
theorem mean_value_theorem(f: Real -> Real, df: Real -> Real, a: Real, b: Real) {
    continuous(f) and a < b and is_derivative_fn(f, df)
    implies exists(c: Real) {
        a < c and c < b and has_derivative_at(f, c, secant_slope(f, a, b))
    }
} by {
    if continuous(f) and a < b and is_derivative_fn(f, df) {
        secant_remainder_continuous_on_closed(f, a, b)
        continuous_on_closed(secant_remainder(f, a, b), a, b)
        secant_remainder_is_derivative_on_open(f, df, a, b)
        is_derivative_on_open(secant_remainder(f, a, b),
            secant_remainder_derivative(df, f, a, b), a, b)
        is_derivative_on_open_imp_differentiable_on_open(secant_remainder(f, a, b),
            secant_remainder_derivative(df, f, a, b), a, b)
        differentiable_on_open(secant_remainder(f, a, b), a, b)
        secant_remainder_endpoints_equal(f, a, b)
        secant_remainder(f, a, b, a) = secant_remainder(f, a, b, b)
        rolle_theorem(secant_remainder(f, a, b), a, b)
        let c: Real satisfy {
            a < c and c < b and has_derivative_at(secant_remainder(f, a, b), c, Real.0)
        }
        a < c and c < b
        has_derivative_at(secant_remainder(f, a, b), c, Real.0)
        is_derivative_fn_at(f, df, c)
        has_derivative_at(f, c, df(c))
        affine_real_has_derivative_at(secant_slope(f, a, b), -(secant_slope(f, a, b)) * a, c)
        has_derivative_at(affine_real(secant_slope(f, a, b), -(secant_slope(f, a, b)) * a), c,
            secant_slope(f, a, b))
        derivative_pointwise_sub(f,
            affine_real(secant_slope(f, a, b), -(secant_slope(f, a, b)) * a),
            c, df(c), secant_slope(f, a, b))
        has_derivative_at(pointwise_add(f,
            pointwise_neg(affine_real(secant_slope(f, a, b), -(secant_slope(f, a, b)) * a))),
            c, df(c) + -secant_slope(f, a, b))
        df(c) + -secant_slope(f, a, b) = df(c) - secant_slope(f, a, b)
        has_derivative_at(pointwise_add(f,
            pointwise_neg(affine_real(secant_slope(f, a, b), -(secant_slope(f, a, b)) * a))),
            c, df(c) - secant_slope(f, a, b))
        secant_remainder_eq_pointwise(f, a, b)
        secant_remainder(f, a, b) = pointwise_add(f, pointwise_neg(
            affine_real(secant_slope(f, a, b), -(secant_slope(f, a, b)) * a)))
        secant_remainder_derivative(df, f, a, b, c) = df(c) - secant_slope(f, a, b)
        function_eq_transport_predicate_rev(
            function(h: Real -> Real) {
                has_derivative_at(h, c, secant_remainder_derivative(df, f, a, b, c))
            },
            secant_remainder(f, a, b),
            pointwise_add(f, pointwise_neg(
                affine_real(secant_slope(f, a, b), -(secant_slope(f, a, b)) * a))))
        has_derivative_at(secant_remainder(f, a, b), c,
            secant_remainder_derivative(df, f, a, b, c))
        has_derivative_at_unique(secant_remainder(f, a, b), c, Real.0,
            secant_remainder_derivative(df, f, a, b, c))
        Real.0 = secant_remainder_derivative(df, f, a, b, c)
        secant_remainder_derivative(df, f, a, b, c) = Real.0
        secant_remainder_derivative(df, f, a, b, c) = df(c) - secant_slope(f, a, b)
        df(c) - secant_slope(f, a, b) = Real.0
        sub_zero_imp_eq(df(c), secant_slope(f, a, b))
        df(c) = secant_slope(f, a, b)
        has_derivative_at(f, c, df(c))
        df(c) = secant_slope(f, a, b)
        has_derivative_at(f, c, secant_slope(f, a, b))
        exists(c2: Real) {
            a < c2 and c2 < b and has_derivative_at(f, c2, secant_slope(f, a, b))
        }
    }
}

/// The mean value theorem, local form: a function continuous on the closed
/// interval [a, b] and differentiable on the open interval (a, b), with df a
/// pointwise derivative there, has a point where its derivative equals the
/// chord slope.
theorem mean_value_theorem_local(f: Real -> Real, df: Real -> Real, a: Real, b: Real) {
    continuous_on_closed(f, a, b) and a < b and is_derivative_on_open(f, df, a, b)
    implies exists(c: Real) {
        a < c and c < b and has_derivative_at(f, c, secant_slope(f, a, b))
    }
} by {
    if continuous_on_closed(f, a, b) and a < b and is_derivative_on_open(f, df, a, b) {
        secant_remainder_continuous_on_closed_local(f, a, b)
        continuous_on_closed(secant_remainder(f, a, b), a, b)
        secant_remainder_is_derivative_on_open_local(f, df, a, b)
        is_derivative_on_open(secant_remainder(f, a, b),
            secant_remainder_derivative(df, f, a, b), a, b)
        is_derivative_on_open_imp_differentiable_on_open(secant_remainder(f, a, b),
            secant_remainder_derivative(df, f, a, b), a, b)
        differentiable_on_open(secant_remainder(f, a, b), a, b)
        secant_remainder_endpoints_equal(f, a, b)
        secant_remainder(f, a, b, a) = secant_remainder(f, a, b, b)
        rolle_theorem(secant_remainder(f, a, b), a, b)
        let c: Real satisfy {
            a < c and c < b and has_derivative_at(secant_remainder(f, a, b), c, Real.0)
        }
        a < c and c < b
        has_derivative_at(secant_remainder(f, a, b), c, Real.0)
        is_derivative_on_open(f, df, a, b) = forall(x: Real) {
            a < x and x < b implies has_derivative_at(f, x, df(x))
        }
        a < c and c < b implies has_derivative_at(f, c, df(c))
        has_derivative_at(f, c, df(c))
        affine_real_has_derivative_at(secant_slope(f, a, b), -(secant_slope(f, a, b)) * a, c)
        has_derivative_at(affine_real(secant_slope(f, a, b), -(secant_slope(f, a, b)) * a), c,
            secant_slope(f, a, b))
        derivative_pointwise_sub(f,
            affine_real(secant_slope(f, a, b), -(secant_slope(f, a, b)) * a),
            c, df(c), secant_slope(f, a, b))
        has_derivative_at(pointwise_add(f,
            pointwise_neg(affine_real(secant_slope(f, a, b), -(secant_slope(f, a, b)) * a))),
            c, df(c) + -secant_slope(f, a, b))
        df(c) + -secant_slope(f, a, b) = df(c) - secant_slope(f, a, b)
        has_derivative_at(pointwise_add(f,
            pointwise_neg(affine_real(secant_slope(f, a, b), -(secant_slope(f, a, b)) * a))),
            c, df(c) - secant_slope(f, a, b))
        secant_remainder_eq_pointwise(f, a, b)
        secant_remainder(f, a, b) = pointwise_add(f, pointwise_neg(
            affine_real(secant_slope(f, a, b), -(secant_slope(f, a, b)) * a)))
        secant_remainder_derivative(df, f, a, b, c) = df(c) - secant_slope(f, a, b)
        function_eq_transport_predicate_rev(
            function(h: Real -> Real) {
                has_derivative_at(h, c, secant_remainder_derivative(df, f, a, b, c))
            },
            secant_remainder(f, a, b),
            pointwise_add(f, pointwise_neg(
                affine_real(secant_slope(f, a, b), -(secant_slope(f, a, b)) * a))))
        has_derivative_at(secant_remainder(f, a, b), c,
            secant_remainder_derivative(df, f, a, b, c))
        has_derivative_at_unique(secant_remainder(f, a, b), c, Real.0,
            secant_remainder_derivative(df, f, a, b, c))
        Real.0 = secant_remainder_derivative(df, f, a, b, c)
        secant_remainder_derivative(df, f, a, b, c) = Real.0
        secant_remainder_derivative(df, f, a, b, c) = df(c) - secant_slope(f, a, b)
        df(c) - secant_slope(f, a, b) = Real.0
        sub_zero_imp_eq(df(c), secant_slope(f, a, b))
        df(c) = secant_slope(f, a, b)
        has_derivative_at(f, c, df(c))
        df(c) = secant_slope(f, a, b)
        has_derivative_at(f, c, secant_slope(f, a, b))
        exists(c2: Real) {
            a < c2 and c2 < b and has_derivative_at(f, c2, secant_slope(f, a, b))
        }
    }
}

/// A function whose derivative is zero everywhere is constant on every interval.
theorem zero_derivative_imp_constant(
    f: Real -> Real, a: Real, b: Real, x: Real
) {
    continuous(f) and a < b and is_derivative_fn(f, constant[Real, Real](Real.0)) and
    closed_interval_set(a, b).contains(x)
    implies f(x) = f(a)
} by {
    if continuous(f) and a < b and is_derivative_fn(f, constant[Real, Real](Real.0)) and
       closed_interval_set(a, b).contains(x) {
        closed_interval_set_lower_le(a, b, x)
        a <= x
        closed_interval_set_le_upper(a, b, x)
        x <= b
        if a < x {
            mean_value_theorem(f, constant[Real, Real](Real.0), a, x)
            let c: Real satisfy {
                a < c and c < x and has_derivative_at(f, c, secant_slope(f, a, x))
            }
            a < c and c < x
            has_derivative_at(f, c, secant_slope(f, a, x))
            is_derivative_fn_at(f, constant[Real, Real](Real.0), c)
            has_derivative_at(f, c, constant[Real, Real](Real.0, c))
            constant[Real, Real](Real.0, c) = Real.0
            has_derivative_at(f, c, Real.0)
            has_derivative_at_unique(f, c, secant_slope(f, a, x), Real.0)
            secant_slope(f, a, x) = Real.0
            secant_slope(f, a, x) = (f(x) - f(a)) / (x - a)
            (f(x) - f(a)) / (x - a) = Real.0
            lt_imp_ne_symm(a, x)
            x != a
            sub_ne_zero_of_ne(x, a)
            x - a != Real.0
            div_mul_cancel_denominator(f(x) - f(a), x - a)
            ((f(x) - f(a)) / (x - a)) * (x - a) = f(x) - f(a)
            ((f(x) - f(a)) / (x - a)) * (x - a) = Real.0 * (x - a)
            mul_zero_left(x - a)
            Real.0 * (x - a) = Real.0
            f(x) - f(a) = Real.0
            sub_zero_imp_eq(f(x), f(a))
            f(x) = f(a)
        } else {
            not a < x
            not_lt_imp_gte[Real](a, x)
            x <= a
            lte_antisymm[Real](a, x)
            a = x
            x = a
            f(x) = f(a)
        }
    }
}

/// A function with a nonnegative derivative everywhere is nondecreasing on every interval.
theorem nonneg_derivative_imp_nondecreasing(
    f: Real -> Real, df: Real -> Real, a: Real, b: Real, x: Real, y: Real
) {
    continuous(f) and a < b and is_derivative_fn(f, df) and
    (forall(z: Real) { Real.0 <= df(z) }) and
    closed_interval_set(a, b).contains(x) and closed_interval_set(a, b).contains(y) and
    x <= y
    implies f(x) <= f(y)
} by {
    if continuous(f) and a < b and is_derivative_fn(f, df) and
       (forall(z: Real) { Real.0 <= df(z) }) and
       closed_interval_set(a, b).contains(x) and closed_interval_set(a, b).contains(y) and
       x <= y {
        if x < y {
            mean_value_theorem(f, df, x, y)
            let c: Real satisfy {
                x < c and c < y and has_derivative_at(f, c, secant_slope(f, x, y))
            }
            x < c and c < y
            has_derivative_at(f, c, secant_slope(f, x, y))
            is_derivative_fn_at(f, df, c)
            has_derivative_at(f, c, df(c))
            has_derivative_at_unique(f, c, secant_slope(f, x, y), df(c))
            secant_slope(f, x, y) = df(c)
            forall(z: Real) { Real.0 <= df(z) }
            Real.0 <= df(c)
            Real.0 <= secant_slope(f, x, y)
            secant_slope(f, x, y) = (f(y) - f(x)) / (y - x)
            Real.0 <= (f(y) - f(x)) / (y - x)
            lt_imp_minus_pos(x, y)
            (y - x).is_positive
            pos_gt_zero(y - x)
            y - x > Real.0
            inverse_of_positive_is_positive[Real](y - x)
            Real.0 < (y - x).inverse
            lt_imp_lte(Real.0, (y - x).inverse)
            Real.0 <= (y - x).inverse
            (f(y) - f(x)) / (y - x) = (f(y) - f(x)) * (y - x).inverse
            Real.0 <= (f(y) - f(x)) * (y - x).inverse
            multiply_inequality_with_nonnegative_element[Real](Real.0,
                (f(y) - f(x)) * (y - x).inverse, y - x)
            Real.0 * (y - x) <= ((f(y) - f(x)) * (y - x).inverse) * (y - x)
            mul_zero_left(y - x)
            Real.0 * (y - x) = Real.0
            Real.0 <= ((f(y) - f(x)) * (y - x).inverse) * (y - x)
            lt_imp_ne_symm(x, y)
            y != x
            sub_ne_zero_of_ne(y, x)
            y - x != Real.0
            div_mul_cancel_denominator(f(y) - f(x), y - x)
            ((f(y) - f(x)) / (y - x)) * (y - x) = f(y) - f(x)
            ((f(y) - f(x)) * (y - x).inverse) * (y - x) =
                ((f(y) - f(x)) / (y - x)) * (y - x)
            ((f(y) - f(x)) * (y - x).inverse) * (y - x) = f(y) - f(x)
            Real.0 <= f(y) - f(x)
            lte_add_right(Real.0, f(y) - f(x), f(x))
            Real.0 + f(x) <= (f(y) - f(x)) + f(x)
            add_zero_left(f(x))
            Real.0 + f(x) = f(x)
            (f(y) - f(x)) + f(x) = f(y)
            f(x) <= f(y)
        } else {
            not x < y
            not_lt_imp_gte[Real](x, y)
            y <= x
            lte_antisymm[Real](x, y)
            x = y
            y = x
            f(y) = f(x)
            lte_refl(f(x))
            f(x) <= f(x)
            f(x) <= f(y)
        }
    }
}
