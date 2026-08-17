/// Square-root facts for absolute values of real numbers.

from real.sqrt import Real, sqrt_square_of_nonneg
from real.real_base import abs_gte_zero, neg_lt_zero, lt_swap_neg, neg_neg,
    lt_add_one
from real.real_ring import mul_one_right, mul_neg_left, mul_neg_right,
    non_neg_imp_zero_lte

numerals Real

/// The square root of one is one.
theorem sqrt_one {
    Real.1.sqrt = Option.some(Real.1)
} by {
    lt_add_one(Real.0)
    Real.0 < Real.0 + Real.1
    Real.0 + Real.1 = Real.1
    Real.0 < Real.1
    Real.1 > Real.0
    Real.1 >= Real.0
    sqrt_square_of_nonneg(Real.1)
    (Real.1 * Real.1).sqrt = Option.some(Real.1)
    mul_one_right(Real.1)
    Real.1 * Real.1 = Real.1
    Real.1.sqrt = Option.some(Real.1)
}

/// The square root of the square of an absolute value is that absolute value.
theorem sqrt_abs_square(x: Real) {
    (x.abs * x.abs).sqrt = Option.some(x.abs)
} by {
    abs_gte_zero(x)
    x.abs >= Real.0
    sqrt_square_of_nonneg(x.abs)
}

/// The square root of a square is the absolute value.
theorem sqrt_square_eq_abs(x: Real) {
    (x * x).sqrt = Option.some(x.abs)
} by {
    if x.is_negative {
        neg_lt_zero(x)
        x < Real.0
        lt_swap_neg(x, Real.0)
        -Real.0 < -x
        -Real.0 = Real.0
        Real.0 < -x
        -x > Real.0
        -x >= Real.0
        (if x.is_negative { -x } else { x }) = x.abs
        (if x.is_negative { -x } else { x }) = -x
        -x = x.abs
        x.abs = -x
        sqrt_square_of_nonneg(-x)
        ((-x) * (-x)).sqrt = Option.some(-x)
        mul_neg_left(x, -x)
        (-x) * (-x) = -(x * (-x))
        mul_neg_right(x, x)
        x * (-x) = -(x * x)
        -(x * (-x)) = -(-(x * x))
        neg_neg(x * x)
        -(-(x * x)) = x * x
        (-x) * (-x) = x * x
        (x * x).sqrt = ((-x) * (-x)).sqrt
        (x * x).sqrt = Option.some(-x)
        (x * x).sqrt = Option.some(x.abs)
    } else {
        not x.is_negative
        non_neg_imp_zero_lte(x)
        Real.0 <= x
        x >= Real.0
        (if x.is_negative { -x } else { x }) = x.abs
        (if x.is_negative { -x } else { x }) = x
        x = x.abs
        x.abs = x
        sqrt_square_of_nonneg(x)
        (x * x).sqrt = Option.some(x)
        (x * x).sqrt = Option.some(x.abs)
    }
}

/// A synonym emphasizing the product of absolute values.
theorem sqrt_abs_mul_abs_eq_abs(x: Real) {
    (x.abs * x.abs).sqrt = Option.some(x.abs)
} by {
    sqrt_abs_square(x)
}
