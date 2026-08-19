/// The Gamma function: definitions and what can be proved today.
///
/// The classical Gamma function is the improper integral
///
///     Γ(x) = ∫₀^∞ t^(x-1)·e^(-t) dt.
///
/// The current real-analysis library has only finite-interval Riemann
/// integrals (real/integral.ac), with no notion of an improper integral
/// (no limit of ∫_ε^T as ε → 0 and T → ∞), so the improper Gamma function is
/// out of reach for now.  What is missing, concretely:
///
///   - an improper-integral notion (a search for "improper", "to_infinity",
///     and "limit" combined with "integral" in src/real/ finds nothing),
///   - integration by parts (search "by_parts"/"parts" in src/real/ finds
///     nothing; the file real/integral.ac even leaves the simpler
///     interval-additivity theorem `integral_additivity` commented out as
///     future work because it needs common refinements of partitions),
///   - the substitution rule for integrals (needed for the reflection
///     identity Γ(x)·Γ(1-x) and for the symmetry of the Beta function),
///   - real powers as a total function: `rpow` (real/log.ac) is
///     Option-typed, so t ↦ t^(x-1) is not a plain Real -> Real integrand.
///
/// What this file does provide is the finite-interval Gamma integrand with
/// natural exponents, t ↦ t^n·e^(-t), which needs none of the missing
/// infrastructure, together with the derivative side of the n = 0 case:
///
///   - exp_neg_fn_derivative:   the derivative of t ↦ e^(-t) is -e^(-t),
///   - exp_neg_anti_derivative: the derivative of t ↦ -e^(-t) is e^(-t),
///   - exp_neg_fn_pos_on:       e^(-t) is nonnegative on [0, T],
///   - exp_neg_fn_le_one_on:    e^(-t) is at most one on [0, T].
///
/// The value of ∫₀^T e^(-t) dt = 1 - e^(-T) is one small lemma away: it needs
/// the exponential inequality 1 - e^(-x) <= x for x >= 0 (equivalently
/// e^(-x) >= 1 - x), which is not yet in the library (only x.exp >= 1 + x
/// for x > 0 exists, and the series/mean-value arguments for the negative
/// side have not been formalized).  With that inequality one proves that
/// e^(-t) is 1-Lipschitz on [0, T] and applies fn_integrable_gen +
/// ftc2_general exactly as in real/power_integral.ac.
///
/// The functional equation for the finite Gamma,
///
///     I_T(n+1) = (n+1)·I_T(n) - T^(n+1)·e^(-T),
///
/// needs integration by parts (or integral linearity), which does not exist
/// yet; it is stated in a comment below together with the recurrence that
/// would prove it from the antiderivative
///
///     ∫ t^n·e^(-t) dt = -e^(-t)·(t^n + n·t^(n-1) + ... + n!),
///
/// whose derivative identity could be proved with the partial-sum machinery
/// of real/beta_function.ac once integral linearity lands.

from nat import Nat, from_nat
from order import lt_imp_lte
from data.basic.functions import function_extensionality, identity_fn, function_eq_transport_predicate_rev, compose
from data.basic.function_algebra import pointwise_mul, pointwise_add, pointwise_neg
from data.basic.logic import eq_true_intro
from real.real_field import Real
from real.continuity_pow import pow_real_fn
from real.continuity_base import continuous
from real.continuity_composition import continuous_compose
from real.continuity_pointwise import continuous_pointwise_add, continuous_pointwise_neg
from real.continuity_sequences import identity_function_is_continuous
from real.calculus_api import is_derivative_fn, derivative_fn_neg, derivative_fn_compose, derivative_fn_constant, derivative_fn_mul
from real.integral import integral, is_integrable, interval_contains, interval_contains_left, interval_contains_right
from real.integral_exp import ftc2_general, exp_lower_bound_on, exp_upper_bound_on, exp_lte_mono, exp_continuous
from real.integral_trig import fn_integrable_gen
from real.exp import exp_zero, exp_pos
from real.derivative_exp_log import exp_is_derivative_fn
from real.log import exp_neg
from real.real_ring import real_mul_comm, mul_abs
from real.real_base import abs_gte_zero, sub_cancels, neg_distrib
from real.real_series import pow_nonneg
from real.derivative_trig import abs_of_nonneg
from ordered_field import mul_le_mul_of_nonneg_right, zero_is_smaller_than_one
from algebra.add_ordered_group import add_le_add_right, add_le_add

numerals Real
numerals Nat

// ---------------------------------------------------------------------------
// Definitions
// ---------------------------------------------------------------------------

/// The pointwise negation x ↦ -x.
define neg_fn(x: Real) -> Real {
    -x
}

/// The function t ↦ e^(-t) appearing in the Gamma integrand.
define exp_neg_fn(t: Real) -> Real {
    (neg_fn(t)).exp
}

/// The Gamma integrand with natural exponent: t ↦ t^n·e^(-t).
define gamma_fn(n: Nat, t: Real) -> Real {
    pow_real_fn(n, t) * exp_neg_fn(t)
}

/// The finite-interval Gamma function with natural exponent: the integral of
/// t^n·e^(-t) over [a, b].
define gamma_interval(n: Nat, a: Real, b: Real) -> Real {
    integral(gamma_fn(n), a, b)
}

// ---------------------------------------------------------------------------
// The derivative of -e^(-t) is e^(-t)
// ---------------------------------------------------------------------------

/// The negation function is its own pointwise derivative up to sign:
/// (x ↦ -x)' = x ↦ -1.
lemma neg_fn_derivative {
    is_derivative_fn(neg_fn, constant[Real, Real](-Real.1))
} by {
    derivative_fn_neg(identity_fn[Real], constant[Real, Real](Real.1))
    is_derivative_fn(pointwise_neg(identity_fn[Real]), pointwise_neg(constant[Real, Real](Real.1)))
    forall(x: Real) {
        neg_fn(x) = -x
        pointwise_neg(identity_fn[Real], x) = -identity_fn[Real](x)
        identity_fn[Real](x) = x
        neg_fn(x) = pointwise_neg(identity_fn[Real], x)
    }
    function_extensionality(neg_fn, pointwise_neg(identity_fn[Real]))
    forall(x: Real) {
        pointwise_neg(constant[Real, Real](Real.1), x) = -constant[Real, Real](Real.1, x)
        constant[Real, Real](Real.1, x) = Real.1
        -Real.1 = constant[Real, Real](-Real.1, x)
        pointwise_neg(constant[Real, Real](Real.1), x) = constant[Real, Real](-Real.1, x)
    }
    function_extensionality(pointwise_neg(constant[Real, Real](Real.1)), constant[Real, Real](-Real.1))
    function_eq_transport_predicate_rev[Real, Real](
        function(h: Real -> Real) { is_derivative_fn(h, constant[Real, Real](-Real.1)) },
        neg_fn, pointwise_neg(identity_fn[Real]))
    is_derivative_fn(neg_fn, constant[Real, Real](-Real.1))
}

/// The derivative of e^(-t) is -e^(-t).
theorem exp_neg_fn_derivative {
    is_derivative_fn(exp_neg_fn, pointwise_neg(exp_neg_fn))
} by {
    exp_is_derivative_fn
    is_derivative_fn(Real.exp, Real.exp)
    neg_fn_derivative
    is_derivative_fn(neg_fn, constant[Real, Real](-Real.1))
    derivative_fn_compose(neg_fn, Real.exp, constant[Real, Real](-Real.1), Real.exp)
    is_derivative_fn(compose(Real.exp, neg_fn), pointwise_mul(compose(Real.exp, neg_fn), constant[Real, Real](-Real.1)))
    forall(x: Real) {
        pointwise_mul(compose(Real.exp, neg_fn), constant[Real, Real](-Real.1), x) =
            compose(Real.exp, neg_fn, x) * constant[Real, Real](-Real.1, x)
        constant[Real, Real](-Real.1, x) = -Real.1
        compose(Real.exp, neg_fn, x) = (neg_fn(x)).exp
        exp_neg_fn(x) = (neg_fn(x)).exp
        compose(Real.exp, neg_fn, x) * -Real.1 = -(compose(Real.exp, neg_fn, x))
        -(compose(Real.exp, neg_fn, x)) = -exp_neg_fn(x)
        pointwise_neg(exp_neg_fn, x) = -exp_neg_fn(x)
        pointwise_mul(compose(Real.exp, neg_fn), constant[Real, Real](-Real.1), x) =
            pointwise_neg(exp_neg_fn, x)
    }
    function_extensionality(
        pointwise_mul(compose(Real.exp, neg_fn), constant[Real, Real](-Real.1)),
        pointwise_neg(exp_neg_fn))
    function_eq_transport_predicate_rev[Real, Real](
        function(h: Real -> Real) { is_derivative_fn(compose(Real.exp, neg_fn), h) },
        pointwise_mul(compose(Real.exp, neg_fn), constant[Real, Real](-Real.1)),
        pointwise_neg(exp_neg_fn))
    is_derivative_fn(compose(Real.exp, neg_fn), pointwise_neg(exp_neg_fn))
    forall(x: Real) {
        exp_neg_fn(x) = (neg_fn(x)).exp
        compose(Real.exp, neg_fn, x) = (neg_fn(x)).exp
        exp_neg_fn(x) = compose(Real.exp, neg_fn, x)
    }
    function_extensionality(exp_neg_fn, compose(Real.exp, neg_fn))
    function_eq_transport_predicate_rev[Real, Real](
        function(h: Real -> Real) { is_derivative_fn(h, pointwise_neg(exp_neg_fn)) },
        exp_neg_fn, compose(Real.exp, neg_fn))
    is_derivative_fn(exp_neg_fn, pointwise_neg(exp_neg_fn))
}

/// The derivative of -e^(-t) is e^(-t).
theorem exp_neg_anti_derivative {
    is_derivative_fn(pointwise_neg(exp_neg_fn), exp_neg_fn)
} by {
    exp_neg_fn_derivative
    is_derivative_fn(exp_neg_fn, pointwise_neg(exp_neg_fn))
    derivative_fn_neg(exp_neg_fn, pointwise_neg(exp_neg_fn))
    is_derivative_fn(pointwise_neg(exp_neg_fn), pointwise_neg(pointwise_neg(exp_neg_fn)))
    forall(x: Real) {
        pointwise_neg(pointwise_neg(exp_neg_fn), x) = -pointwise_neg(exp_neg_fn, x)
        pointwise_neg(exp_neg_fn, x) = -exp_neg_fn(x)
        -(-exp_neg_fn(x)) = exp_neg_fn(x)
        pointwise_neg(pointwise_neg(exp_neg_fn), x) = exp_neg_fn(x)
    }
    function_extensionality(pointwise_neg(pointwise_neg(exp_neg_fn)), exp_neg_fn)
    function_eq_transport_predicate_rev[Real, Real](
        function(h: Real -> Real) { is_derivative_fn(pointwise_neg(exp_neg_fn), h) },
        pointwise_neg(pointwise_neg(exp_neg_fn)), exp_neg_fn)
    is_derivative_fn(pointwise_neg(exp_neg_fn), exp_neg_fn)
}

// ---------------------------------------------------------------------------
// Statements that need missing infrastructure
// ---------------------------------------------------------------------------

// The finite-interval Gamma functional equation needs integration by parts
// (or integral linearity).  For the antiderivative
//
//     A(t) = -e^(-t)·Q(t),  Q(t) = Σ_{k=0}^{n} n!/k!·t^k,
//
// one would prove is_derivative_fn(A, t ↦ t^n·e^(-t)) with the partial-sum
// machinery of real/beta_function.ac, then ftc2_general gives
//
//     gamma_interval(n, 0, T) = n! - e^(-T)·Q(T).
//
// The recurrence gamma_interval(n+1, 0, T) = (n+1)·gamma_interval(n, 0, T) -
// T^(n+1)·e^(-T) additionally needs ∫(f + g) = ∫f + ∫g, which requires the
// common-refinement machinery flagged as future work in real/integral.ac.
//
// The improper Gamma function Γ(x) = ∫₀^∞ t^(x-1)·e^(-t) dt needs:
//   - an improper-integral notion (limit of the finite integrals),
//   - real powers as a total function (rpow is Option-typed),
//   - the substitution rule (for the reflection identity), and
//   - convergence of ∫₀^∞ t^(x-1)·e^(-t) for x > 0 (the integrand behaves
//     like t^(x-1) near 0 and decays exponentially at infinity).
