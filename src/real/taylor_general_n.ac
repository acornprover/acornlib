/// The general n-th order Taylor theorem with Lagrange remainder.
///
/// Taylor's theorem of order n says that the value of an (n + 1)-times
/// differentiable function at b is its n-th order Taylor polynomial at a plus
/// a Lagrange remainder f^(n + 1)(c) (b - a)^(n + 1) / (n + 1)! for some
/// interior point c.  This file proves the statement for every order n at
/// once, generalizing the order zero, one, two and three instances of
/// taylor_general.ac.
///
/// As in taylor_general.ac, the n-th iterated derivative is represented by a
/// chain dfs of derivative functions: dfs(0) is f itself and dfs(k + 1) is a
/// derivative function of dfs(k), and the chain of length n + 1 is the
/// hypothesis `is_derivative_chain(f, dfs, n + 1)`.
///
/// The classical proof applies Rolle's theorem to the auxiliary function
///   g(x) = f(b) - sum_{k=0}^{n} f^(k)(x) (b - x)^k / k! - M (b - x)^(n + 1)
/// with M chosen so that g(a) = 0.  The derivative of g telescopes: the
/// k-th term contributes f^(k + 1)(x) (b - x)^k / k! and the shifted term
/// -f^(k)(x) k (b - x)^(k - 1) / k! cancels the (k - 1)-st term, leaving
///   g'(x) = (b - x)^n ((n + 1) M - f^(n + 1)(x) / n!).
/// Rolle's theorem then yields c with g'(c) = 0, so M = f^(n + 1)(c) / (n + 1)!,
/// and g(a) = 0 is the Taylor formula.  The telescoping is carried out with
/// the partial-sum lemmas of src/list/list_sum.ac, and the differentiability
/// of g is built up from the pointwise derivative rules applied to each term.

from order import lt_trans, lt_imp_ne_symm, lt_imp_ne, lt_imp_lte, lt_or_lte
from order_set import closed_interval_set
from nat import Nat, from_nat, from_nat_zero, from_nat_one, from_nat_add, from_nat_mul, pow_zero, pow_one, one_pow, pow_add, factorial_zero, factorial_one, factorial_step, alt_induction, zero_or_suc, suc_sub_one, sub_self, sub_zero, add_sub, sub_lt, lt_imp_lt_suc, lt_suc, semiring_zero_pow, alt_suc_ne_zero, lt_suc_right, not_lt_zero
from rat import Rat
from list import partial, partial_split_last, partial_shift_suc, partial_zero, partial_one, partial_pointwise_eq, partial_scalar_mul, partial_add
from real.continuity_base import Real, continuous, continuous_at
from real.derivative_basic import has_derivative_at, has_derivative_at_unique, sub_ne_zero_of_ne, constant_has_derivative_at, identity_has_derivative_at, forall_elim
from real.derivative_rules import derivative_pointwise_sub, derivative_pointwise_add, derivative_pointwise_neg
from real.derivative_linear import derivative_pointwise_const_mul
from real.derivative_product import derivative_pointwise_mul
from real.derivative_chain import derivative_compose
from real.derivative_affine_named import affine_real_has_derivative_at
from real.derivative_continuity import div_mul_cancel_denominator
from real.calculus_api import is_derivative_fn, is_derivative_fn_at, is_derivative_fn_iff, is_derivative_fn_imp_continuous_at
from real.continuity_affine import affine_real
from real.real_base import add_comm, add_assoc, add_zero_right, add_zero_left, add_neg_eq_zero, neg_zero, neg_distrib, neg_neg
from real.real_ring import mul_zero_left, mul_zero_right, real_mul_comm, mul_assoc, mul_one_right, mul_one_left, mul_distrib_right, mul_distrib_left, mul_neg_right, mul_neg_left, from_nat_is_from_rat
from real.real_field import mul_inverse, mul_left_cancel
from real.real_seq import sub_zero_imp_eq
from real.mean_value import continuous_on_closed, differentiable_on_open, is_derivative_on_open, is_derivative_on_open_imp_differentiable_on_open, rolle_theorem
from real.taylor import is_derivative_fn_imp_is_derivative_on_open, taylor2_sub_add_cancel
from real.exp import factorial_pos, factorial_suc_real, pow_suc, suc_pos, exp_term, exp_zero, exp_term_partial_converges, exp_increasing
from real.derivative_exp_log import exp_is_derivative_fn
from real.real_seq import converges_to, limit, converges_imp_converges_to, converges
from real.taylor_general import taylor_poly, taylor_term, taylor_remainder, iterated_derivative, is_derivative_chain, from_nat_factorial_zero_real, real_div_one, taylor_chain1, taylor_chain2, taylor_chain3, taylor_chain4, taylor_chain1_zero, taylor_chain1_suc, taylor_chain2_zero, taylor_chain2_one, taylor_chain2_suc_suc, taylor_chain3_zero, taylor_chain3_one, taylor_chain3_two, taylor_chain3_three, taylor_chain4_zero, taylor_chain4_one, taylor_chain4_two, taylor_chain4_three, taylor_chain4_four
from algebra.field.field import mul_not_zero, field_square_nonzero, inverse_not_zero, pow_not_zero, inverse_dist
from data.basic.function_algebra import pointwise_add, pointwise_neg, pointwise_mul
from algebra.semigroup import mul_fn
from algebra.add_semigroup import add_fn
from data.basic.functions import identity_fn, compose, function_extensionality, function_eq_transport_predicate_rev

numerals Real

/// The k-th power function x ↦ x^k, built by repeated pointwise multiplication.
define pow_fn(k: Nat) -> Real -> Real {
    match k {
        Nat.zero { constant[Real, Real](Real.1) }
        Nat.suc(m) { pointwise_mul(identity_fn[Real], pow_fn(m)) }
    }
}

/// The zero-th power function is the constant one function.
theorem pow_fn_zero_eq {
    pow_fn(Nat.0) = constant[Real, Real](Real.1)
} by {
    match Nat.0 {
        Nat.zero {
            pow_fn(Nat.0) = constant[Real, Real](Real.1)
        }
        Nat.suc(k) {
            false
        }
    }
}

/// The (k + 1)-st power function is the identity times the k-th power function.
theorem pow_fn_suc_eq(k: Nat) {
    pow_fn(k.suc) = pointwise_mul(identity_fn[Real], pow_fn(k))
} by {
    match k.suc {
        Nat.zero {
            false
        }
        Nat.suc(m) {
            m = k
            pow_fn(k.suc) = pointwise_mul(identity_fn[Real], pow_fn(k))
        }
    }
}

/// The value of the k-th power function at z is z^k.
theorem pow_fn_value(n: Nat, z: Real) {
    pow_fn(n, z) = z.pow(n)
} by {
    define p(i: Nat) -> Bool {
        forall(y: Real) { pow_fn(i, y) = y.pow(i) }
    }
    forall(x: Real) {
        pow_fn_zero_eq
        pow_fn(Nat.0) = constant[Real, Real](Real.1)
        pow_fn(Nat.0, x) = constant[Real, Real](Real.1, x)
        constant[Real, Real](Real.1, x) = Real.1
        pow_zero[Real](x)
        x.pow(Nat.0) = Real.1
        pow_fn(Nat.0, x) = x.pow(Nat.0)
    }
    p(Nat.0)
    forall(k: Nat) {
        if p(k) {
            forall(x: Real) {
                pow_fn_suc_eq(k)
                pow_fn(k.suc) = pointwise_mul(identity_fn[Real], pow_fn(k))
                pow_fn(k.suc, x) = pointwise_mul(identity_fn[Real], pow_fn(k), x)
                pointwise_mul(identity_fn[Real], pow_fn(k), x) =
                    identity_fn[Real](x) * pow_fn(k, x)
                identity_fn[Real](x) = x
                pow_fn(k, x) = x.pow(k)
                identity_fn[Real](x) * pow_fn(k, x) = x * x.pow(k)
                pow_suc(x, k)
                x.pow(k.suc) = x * x.pow(k)
                x * x.pow(k) = x.pow(k.suc)
                identity_fn[Real](x) * pow_fn(k, x) = x.pow(k.suc)
                pow_fn(k.suc, x) = x.pow(k.suc)
            }
            p(k.suc)
        }
    }
    p(Nat.0) and forall(k: Nat) { p(k) implies p(k.suc) }
    alt_induction(p)
    forall(k: Nat) { p(k) }
    p(n)
    forall_elim[Real](function(y: Real) { pow_fn(n, y) = y.pow(n) }, z)
    pow_fn(n, z) = z.pow(n)
}

/// The derivative of the (m + 1)-st power function is (m + 1) x^m.
theorem pow_fn_has_derivative_at_suc(q: Nat, x0: Real) {
    has_derivative_at(pow_fn(q.suc), x0, from_nat[Real](q.suc) * x0.pow(q))
} by {
    define p(i: Nat) -> Bool {
        has_derivative_at(pow_fn(i.suc), x0, from_nat[Real](i.suc) * x0.pow(i))
    }
    identity_has_derivative_at(x0)
    has_derivative_at(identity_fn[Real], x0, Real.1)
    constant_has_derivative_at(Real.1, x0)
    has_derivative_at(constant[Real, Real](Real.1), x0, Real.0)
    derivative_pointwise_mul(identity_fn[Real], constant[Real, Real](Real.1), x0, Real.1, Real.0)
    has_derivative_at(pointwise_mul(identity_fn[Real], constant[Real, Real](Real.1)), x0,
        identity_fn[Real](x0) * Real.0 + constant[Real, Real](Real.1, x0) * Real.1)
    identity_fn[Real](x0) = x0
    mul_zero_right(x0)
    x0 * Real.0 = Real.0
    constant[Real, Real](Real.1, x0) = Real.1
    mul_one_right(Real.1)
    Real.1 * Real.1 = Real.1
    constant[Real, Real](Real.1, x0) * Real.1 = Real.1
    add_zero_left(Real.1)
    Real.0 + Real.1 = Real.1
    identity_fn[Real](x0) * Real.0 + constant[Real, Real](Real.1, x0) * Real.1 = Real.1
    has_derivative_at(pointwise_mul(identity_fn[Real], constant[Real, Real](Real.1)), x0, Real.1)
    pow_fn_zero_eq
    pow_fn(Nat.0) = constant[Real, Real](Real.1)
    pow_fn_suc_eq(Nat.0)
    pow_fn(Nat.0.suc) = pointwise_mul(identity_fn[Real], pow_fn(Nat.0))
    pow_fn(Nat.0.suc) = pointwise_mul(identity_fn[Real], constant[Real, Real](Real.1))
    function_eq_transport_predicate_rev(
        function(h: Real -> Real) {
            has_derivative_at(h, x0, Real.1)
        },
        pow_fn(Nat.0.suc), pointwise_mul(identity_fn[Real], constant[Real, Real](Real.1)))
    has_derivative_at(pow_fn(Nat.0.suc), x0, Real.1)
    from_nat_add[Real](Nat.0, Nat.1)
    from_nat[Real](Nat.0 + Nat.1) = from_nat[Real](Nat.0) + from_nat[Real](Nat.1)
    Nat.0 + Nat.1 = Nat.1
    from_nat[Real](Nat.1) = from_nat[Real](Nat.0) + from_nat[Real](Nat.1)
    from_nat_zero[Real]
    from_nat[Real](Nat.0) = Real.0
    from_nat_one[Real]
    from_nat[Real](Nat.1) = Real.1
    from_nat[Real](Nat.0) + from_nat[Real](Nat.1) = Real.0 + Real.1
    add_zero_left(Real.1)
    Real.0 + Real.1 = Real.1
    from_nat[Real](Nat.1) = Real.1
    Real.1 = from_nat[Real](Nat.1)
    Nat.1 = Nat.0.suc
    from_nat[Real](Nat.0.suc) = Real.1
    pow_zero[Real](x0)
    x0.pow(Nat.0) = Real.1
    from_nat[Real](Nat.0.suc) * x0.pow(Nat.0) = Real.1 * x0.pow(Nat.0)
    mul_one_left(x0.pow(Nat.0))
    Real.1 * x0.pow(Nat.0) = x0.pow(Nat.0)
    from_nat[Real](Nat.0.suc) * x0.pow(Nat.0) = x0.pow(Nat.0)
    x0.pow(Nat.0) = from_nat[Real](Nat.0.suc) * x0.pow(Nat.0)
    Real.1 = from_nat[Real](Nat.0.suc) * x0.pow(Nat.0)
    has_derivative_at(pow_fn(Nat.0.suc), x0, from_nat[Real](Nat.0.suc) * x0.pow(Nat.0))
    p(Nat.0)
    forall(m: Nat) {
        if p(m) {
            identity_has_derivative_at(x0)
            has_derivative_at(identity_fn[Real], x0, Real.1)
            has_derivative_at(pow_fn(m.suc), x0, from_nat[Real](m.suc) * x0.pow(m))
            derivative_pointwise_mul(identity_fn[Real], pow_fn(m.suc), x0, Real.1,
                from_nat[Real](m.suc) * x0.pow(m))
            has_derivative_at(pointwise_mul(identity_fn[Real], pow_fn(m.suc)), x0,
                identity_fn[Real](x0) * (from_nat[Real](m.suc) * x0.pow(m)) +
                    pow_fn(m.suc, x0) * Real.1)
            identity_fn[Real](x0) = x0
            mul_assoc(from_nat[Real](m.suc), x0.pow(m), Real.1)
            (from_nat[Real](m.suc) * x0.pow(m)) * Real.1 =
                from_nat[Real](m.suc) * (x0.pow(m) * Real.1)
            mul_one_right(x0.pow(m))
            x0.pow(m) * Real.1 = x0.pow(m)
            from_nat[Real](m.suc) * (x0.pow(m) * Real.1) =
                from_nat[Real](m.suc) * x0.pow(m)
            (from_nat[Real](m.suc) * x0.pow(m)) * Real.1 =
                from_nat[Real](m.suc) * x0.pow(m)
            pow_suc(x0, m)
            x0.pow(m.suc) = x0 * x0.pow(m)
            real_mul_comm(x0, x0.pow(m))
            x0 * x0.pow(m) = x0.pow(m) * x0
            real_mul_comm(x0.pow(m), x0)
            x0.pow(m) * x0 = x0 * x0.pow(m)
            x0.pow(m.suc) = x0.pow(m) * x0
            real_mul_comm(x0, from_nat[Real](m.suc))
            x0 * from_nat[Real](m.suc) = from_nat[Real](m.suc) * x0
            mul_assoc(x0, from_nat[Real](m.suc), x0.pow(m))
            x0 * (from_nat[Real](m.suc) * x0.pow(m)) =
                (x0 * from_nat[Real](m.suc)) * x0.pow(m)
            real_mul_comm(x0, from_nat[Real](m.suc))
            (x0 * from_nat[Real](m.suc)) * x0.pow(m) =
                (from_nat[Real](m.suc) * x0) * x0.pow(m)
            mul_assoc(from_nat[Real](m.suc), x0, x0.pow(m))
            (from_nat[Real](m.suc) * x0) * x0.pow(m) =
                from_nat[Real](m.suc) * (x0 * x0.pow(m))
            from_nat[Real](m.suc) * (x0 * x0.pow(m)) =
                from_nat[Real](m.suc) * x0.pow(m.suc)
            x0 * (from_nat[Real](m.suc) * x0.pow(m)) =
                from_nat[Real](m.suc) * x0.pow(m.suc)
            pow_fn_value(m.suc, x0)
            pow_fn(m.suc, x0) = x0.pow(m.suc)
            pow_fn(m.suc, x0) * Real.1 = x0.pow(m.suc) * Real.1
            mul_one_right(x0.pow(m.suc))
            x0.pow(m.suc) * Real.1 = x0.pow(m.suc)
            pow_fn(m.suc, x0) * Real.1 = x0.pow(m.suc)
            identity_fn[Real](x0) * (from_nat[Real](m.suc) * x0.pow(m)) +
                pow_fn(m.suc, x0) * Real.1 =
                from_nat[Real](m.suc) * x0.pow(m.suc) + x0.pow(m.suc)
            mul_distrib_right(x0.pow(m.suc), from_nat[Real](m.suc), Real.1)
            x0.pow(m.suc) * (from_nat[Real](m.suc) + Real.1) =
                x0.pow(m.suc) * from_nat[Real](m.suc) + x0.pow(m.suc) * Real.1
            real_mul_comm(x0.pow(m.suc), from_nat[Real](m.suc))
            x0.pow(m.suc) * from_nat[Real](m.suc) =
                from_nat[Real](m.suc) * x0.pow(m.suc)
            mul_one_right(x0.pow(m.suc))
            x0.pow(m.suc) * Real.1 = x0.pow(m.suc)
            x0.pow(m.suc) * (from_nat[Real](m.suc) + Real.1) =
                from_nat[Real](m.suc) * x0.pow(m.suc) + x0.pow(m.suc)
            add_comm(from_nat[Real](m.suc) * x0.pow(m.suc), x0.pow(m.suc))
            from_nat[Real](m.suc) * x0.pow(m.suc) + x0.pow(m.suc) =
                x0.pow(m.suc) + from_nat[Real](m.suc) * x0.pow(m.suc)
            identity_fn[Real](x0) * (from_nat[Real](m.suc) * x0.pow(m)) +
                pow_fn(m.suc, x0) * Real.1 =
                x0.pow(m.suc) * (from_nat[Real](m.suc) + Real.1)
            from_nat_add[Real](m.suc, Nat.1)
            from_nat[Real](m.suc + Nat.1) =
                from_nat[Real](m.suc) + from_nat[Real](Nat.1)
            from_nat_one[Real]
            from_nat[Real](Nat.1) = Real.1
            from_nat[Real](m.suc + Nat.1) =
                from_nat[Real](m.suc) + Real.1
            m.suc + Nat.1 = m.suc.suc
            from_nat[Real](m.suc.suc) =
                from_nat[Real](m.suc) + Real.1
            x0.pow(m.suc) * (from_nat[Real](m.suc) + Real.1) =
                x0.pow(m.suc) * from_nat[Real](m.suc.suc)
            real_mul_comm(x0.pow(m.suc), from_nat[Real](m.suc.suc))
            x0.pow(m.suc) * from_nat[Real](m.suc.suc) =
                from_nat[Real](m.suc.suc) * x0.pow(m.suc)
            identity_fn[Real](x0) * (from_nat[Real](m.suc) * x0.pow(m)) +
                pow_fn(m.suc, x0) * Real.1 =
                from_nat[Real](m.suc.suc) * x0.pow(m.suc)
            has_derivative_at(pointwise_mul(identity_fn[Real], pow_fn(m.suc)), x0,
                from_nat[Real](m.suc.suc) * x0.pow(m.suc))
            pow_fn_suc_eq(m.suc)
            pow_fn(m.suc.suc) = pointwise_mul(identity_fn[Real], pow_fn(m.suc))
            function_eq_transport_predicate_rev(
                function(h: Real -> Real) {
                    has_derivative_at(h, x0, from_nat[Real](m.suc.suc) * x0.pow(m.suc))
                },
                pow_fn(m.suc.suc), pointwise_mul(identity_fn[Real], pow_fn(m.suc)))
            has_derivative_at(pow_fn(m.suc.suc), x0,
                from_nat[Real](m.suc.suc) * x0.pow(m.suc))
            p(m.suc)
        }
    }
    p(Nat.0) and forall(m: Nat) { p(m) implies p(m.suc) }
    alt_induction(p)
    forall(m: Nat) { p(m) }
    p(q)
    has_derivative_at(pow_fn(q.suc), x0, from_nat[Real](q.suc) * x0.pow(q))
}

/// The shifted power function x ↦ (b - x)^k.
define taylor_shift_pow(b: Real, k: Nat) -> Real -> Real {
    compose(pow_fn(k), affine_real(-Real.1, b))
}

/// The value of the shifted power function at x is (b - x)^k.
theorem taylor_shift_pow_value(b: Real, k: Nat, x: Real) {
    taylor_shift_pow(b, k, x) = (b - x).pow(k)
} by {
    taylor_shift_pow(b, k, x) = compose(pow_fn(k), affine_real(-Real.1, b), x)
    compose(pow_fn(k), affine_real(-Real.1, b), x) =
        pow_fn(k, affine_real(-Real.1, b, x))
    affine_real(-Real.1, b, x) = -Real.1 * x + b
    mul_neg_left(Real.1, x)
    -Real.1 * x = -(Real.1 * x)
    mul_one_left(x)
    Real.1 * x = x
    -(Real.1 * x) = -x
    -Real.1 * x = -x
    -x + b = b - x
    affine_real(-Real.1, b, x) = b - x
    pow_fn_value(k, b - x)
    pow_fn(k, b - x) = (b - x).pow(k)
    pow_fn(k, affine_real(-Real.1, b, x)) = (b - x).pow(k)
    taylor_shift_pow(b, k, x) = (b - x).pow(k)
}

/// The derivative of x ↦ (b - x)^(m + 1) is -(m + 1) (b - x)^m.
theorem taylor_shift_pow_suc_has_derivative_at(b: Real, m: Nat, x0: Real) {
    has_derivative_at(taylor_shift_pow(b, m.suc), x0,
        -(from_nat[Real](m.suc)) * (b - x0).pow(m))
} by {
    affine_real_has_derivative_at(-Real.1, b, x0)
    has_derivative_at(affine_real(-Real.1, b), x0, -Real.1)
    pow_fn_has_derivative_at_suc(m, affine_real(-Real.1, b, x0))
    has_derivative_at(pow_fn(m.suc), affine_real(-Real.1, b, x0),
        from_nat[Real](m.suc) * affine_real(-Real.1, b, x0).pow(m))
    affine_real(-Real.1, b, x0) = -Real.1 * x0 + b
    mul_neg_left(Real.1, x0)
    -Real.1 * x0 = -(Real.1 * x0)
    mul_one_left(x0)
    Real.1 * x0 = x0
    -(Real.1 * x0) = -x0
    -Real.1 * x0 = -x0
    -x0 + b = b - x0
    affine_real(-Real.1, b, x0) = b - x0
    from_nat[Real](m.suc) * affine_real(-Real.1, b, x0).pow(m) =
        from_nat[Real](m.suc) * (b - x0).pow(m)
    has_derivative_at(pow_fn(m.suc), affine_real(-Real.1, b, x0),
        from_nat[Real](m.suc) * (b - x0).pow(m))
    derivative_compose(pow_fn(m.suc), affine_real(-Real.1, b), x0, -Real.1,
        from_nat[Real](m.suc) * (b - x0).pow(m))
    has_derivative_at(compose(pow_fn(m.suc), affine_real(-Real.1, b)), x0,
        (from_nat[Real](m.suc) * (b - x0).pow(m)) * -Real.1)
    mul_neg_right(from_nat[Real](m.suc) * (b - x0).pow(m), Real.1)
    (from_nat[Real](m.suc) * (b - x0).pow(m)) * -Real.1 =
        -(from_nat[Real](m.suc) * (b - x0).pow(m))
    mul_neg_left(from_nat[Real](m.suc), (b - x0).pow(m))
    -(from_nat[Real](m.suc) * (b - x0).pow(m)) =
        -(from_nat[Real](m.suc)) * (b - x0).pow(m)
    (from_nat[Real](m.suc) * (b - x0).pow(m)) * -Real.1 =
        -(from_nat[Real](m.suc)) * (b - x0).pow(m)
    has_derivative_at(compose(pow_fn(m.suc), affine_real(-Real.1, b)), x0,
        -(from_nat[Real](m.suc)) * (b - x0).pow(m))
    taylor_shift_pow(b, m.suc) = compose(pow_fn(m.suc), affine_real(-Real.1, b))
    function_eq_transport_predicate_rev(
        function(h: Real -> Real) {
            has_derivative_at(h, x0, -(from_nat[Real](m.suc)) * (b - x0).pow(m))
        },
        taylor_shift_pow(b, m.suc), compose(pow_fn(m.suc), affine_real(-Real.1, b)))
    has_derivative_at(taylor_shift_pow(b, m.suc), x0,
        -(from_nat[Real](m.suc)) * (b - x0).pow(m))
}

/// The derivative of x ↦ (b - x)^0 is zero.
theorem taylor_shift_pow_zero_has_derivative_at(b: Real, x0: Real) {
    has_derivative_at(taylor_shift_pow(b, Nat.0), x0, Real.0)
} by {
    affine_real_has_derivative_at(-Real.1, b, x0)
    has_derivative_at(affine_real(-Real.1, b), x0, -Real.1)
    constant_has_derivative_at(Real.1, affine_real(-Real.1, b, x0))
    has_derivative_at(constant[Real, Real](Real.1), affine_real(-Real.1, b, x0), Real.0)
    derivative_compose(constant[Real, Real](Real.1), affine_real(-Real.1, b), x0,
        -Real.1, Real.0)
    has_derivative_at(compose(constant[Real, Real](Real.1), affine_real(-Real.1, b)), x0,
        Real.0 * -Real.1)
    mul_zero_left(-Real.1)
    Real.0 * -Real.1 = Real.0
    has_derivative_at(compose(constant[Real, Real](Real.1), affine_real(-Real.1, b)), x0,
        Real.0)
    pow_fn_zero_eq
    pow_fn(Nat.0) = constant[Real, Real](Real.1)
    compose(pow_fn(Nat.0), affine_real(-Real.1, b)) =
        compose(constant[Real, Real](Real.1), affine_real(-Real.1, b))
    function_eq_transport_predicate_rev(
        function(h: Real -> Real) {
            has_derivative_at(h, x0, Real.0)
        },
        compose(pow_fn(Nat.0), affine_real(-Real.1, b)),
        compose(constant[Real, Real](Real.1), affine_real(-Real.1, b)))
    has_derivative_at(compose(pow_fn(Nat.0), affine_real(-Real.1, b)), x0, Real.0)
    taylor_shift_pow(b, Nat.0) = compose(pow_fn(Nat.0), affine_real(-Real.1, b))
    function_eq_transport_predicate_rev(
        function(h: Real -> Real) {
            has_derivative_at(h, x0, Real.0)
        },
        taylor_shift_pow(b, Nat.0), compose(pow_fn(Nat.0), affine_real(-Real.1, b)))
    has_derivative_at(taylor_shift_pow(b, Nat.0), x0, Real.0)
}

/// The derivative of x ↦ (b - x)^k is -k (b - x)^(k - 1), for every k.
theorem taylor_shift_pow_has_derivative_at(b: Real, k: Nat, x0: Real) {
    has_derivative_at(taylor_shift_pow(b, k), x0,
        -(from_nat[Real](k)) * (b - x0).pow(k - Nat.1))
} by {
    zero_or_suc(k)
    if k = Nat.0 {
        taylor_shift_pow_zero_has_derivative_at(b, x0)
        has_derivative_at(taylor_shift_pow(b, Nat.0), x0, Real.0)
        from_nat_zero[Real]
        from_nat[Real](Nat.0) = Real.0
        sub_lt(Nat.0, Nat.1)
        Nat.0 - Nat.1 = Nat.0
        from_nat[Real](Nat.0) * (b - x0).pow(Nat.0 - Nat.1) =
            Real.0 * (b - x0).pow(Nat.0)
        mul_zero_left((b - x0).pow(Nat.0))
        Real.0 * (b - x0).pow(Nat.0) = Real.0
        from_nat[Real](Nat.0) * (b - x0).pow(Nat.0 - Nat.1) = Real.0
        neg_zero
        -Real.0 = Real.0
        -(from_nat[Real](Nat.0)) * (b - x0).pow(Nat.0 - Nat.1) = Real.0
        Real.0 = -(from_nat[Real](Nat.0)) * (b - x0).pow(Nat.0 - Nat.1)
        has_derivative_at(taylor_shift_pow(b, Nat.0), x0,
            -(from_nat[Real](Nat.0)) * (b - x0).pow(Nat.0 - Nat.1))
        has_derivative_at(taylor_shift_pow(b, k), x0,
            -(from_nat[Real](k)) * (b - x0).pow(k - Nat.1))
    } else {
        let m: Nat satisfy { k = m.suc }
        taylor_shift_pow_suc_has_derivative_at(b, m, x0)
        has_derivative_at(taylor_shift_pow(b, m.suc), x0,
            -(from_nat[Real](m.suc)) * (b - x0).pow(m))
        suc_sub_one(m)
        m.suc - Nat.1 = m
        (b - x0).pow(m) = (b - x0).pow(m.suc - Nat.1)
        -(from_nat[Real](m.suc)) * (b - x0).pow(m) =
            -(from_nat[Real](m.suc)) * (b - x0).pow(m.suc - Nat.1)
        has_derivative_at(taylor_shift_pow(b, m.suc), x0,
            -(from_nat[Real](m.suc)) * (b - x0).pow(m.suc - Nat.1))
        has_derivative_at(taylor_shift_pow(b, k), x0,
            -(from_nat[Real](k)) * (b - x0).pow(k - Nat.1))
    }
}

/// The k-th term of the backward Taylor expansion of f around b, evaluated at
/// x: the value of the k-th derivative at x times (b - x)^k divided by k!.
define taylor_bt_term(dfs: Nat -> Real -> Real, b: Real, x: Real, k: Nat) -> Real {
    (dfs(k, x) / from_nat[Real](k.factorial)) * (b - x).pow(k)
}

/// The derivative value of the k-th backward Taylor term at x: the value of
/// the (k + 1)-st derivative at x times (b - x)^k divided by k!, minus the
/// value of the k-th derivative at x times k (b - x)^(k - 1) divided by k!.
define taylor_bt_term_deriv(dfs: Nat -> Real -> Real, b: Real, x: Real, k: Nat) -> Real {
    (dfs(k.suc, x) / from_nat[Real](k.factorial)) * (b - x).pow(k) -
        (dfs(k, x) / from_nat[Real](k.factorial)) * from_nat[Real](k) * (b - x).pow(k - Nat.1)
}

/// The k-th backward Taylor term as a function of x.
define taylor_bt_term_fn(dfs: Nat -> Real -> Real, b: Real, k: Nat) -> Real -> Real {
    pointwise_mul(
        pointwise_mul(constant[Real, Real](from_nat[Real](k.factorial).inverse), dfs(k)),
        taylor_shift_pow(b, k))
}

/// The value of the function form of the k-th backward Taylor term at x is the
/// k-th backward Taylor term at x.
theorem taylor_bt_term_fn_value(dfs: Nat -> Real -> Real, b: Real, k: Nat, x: Real) {
    taylor_bt_term_fn(dfs, b, k, x) = taylor_bt_term(dfs, b, x, k)
} by {
    taylor_bt_term_fn(dfs, b, k, x) = pointwise_mul(
        pointwise_mul(constant[Real, Real](from_nat[Real](k.factorial).inverse), dfs(k)),
        taylor_shift_pow(b, k), x)
    pointwise_mul(pointwise_mul(constant[Real, Real](from_nat[Real](k.factorial).inverse), dfs(k)),
        taylor_shift_pow(b, k), x) =
        pointwise_mul(constant[Real, Real](from_nat[Real](k.factorial).inverse), dfs(k), x) *
            taylor_shift_pow(b, k, x)
    pointwise_mul(constant[Real, Real](from_nat[Real](k.factorial).inverse), dfs(k), x) =
        constant[Real, Real](from_nat[Real](k.factorial).inverse, x) * dfs(k, x)
    constant[Real, Real](from_nat[Real](k.factorial).inverse, x) =
        from_nat[Real](k.factorial).inverse
    pointwise_mul(constant[Real, Real](from_nat[Real](k.factorial).inverse), dfs(k), x) =
        from_nat[Real](k.factorial).inverse * dfs(k, x)
    taylor_shift_pow_value(b, k, x)
    taylor_shift_pow(b, k, x) = (b - x).pow(k)
    pointwise_mul(pointwise_mul(constant[Real, Real](from_nat[Real](k.factorial).inverse), dfs(k)),
        taylor_shift_pow(b, k), x) =
        (from_nat[Real](k.factorial).inverse * dfs(k, x)) * (b - x).pow(k)
    taylor_bt_term(dfs, b, x, k) = (dfs(k, x) / from_nat[Real](k.factorial)) * (b - x).pow(k)
    dfs(k, x) / from_nat[Real](k.factorial) =
        dfs(k, x) * from_nat[Real](k.factorial).inverse
    (dfs(k, x) / from_nat[Real](k.factorial)) * (b - x).pow(k) =
        (dfs(k, x) * from_nat[Real](k.factorial).inverse) * (b - x).pow(k)
    real_mul_comm(dfs(k, x), from_nat[Real](k.factorial).inverse)
    dfs(k, x) * from_nat[Real](k.factorial).inverse =
        from_nat[Real](k.factorial).inverse * dfs(k, x)
    (dfs(k, x) * from_nat[Real](k.factorial).inverse) * (b - x).pow(k) =
        (from_nat[Real](k.factorial).inverse * dfs(k, x)) * (b - x).pow(k)
    taylor_bt_term(dfs, b, x, k) =
        (from_nat[Real](k.factorial).inverse * dfs(k, x)) * (b - x).pow(k)
    taylor_bt_term_fn(dfs, b, k, x) = taylor_bt_term(dfs, b, x, k)
}

/// The k-th backward Taylor term is differentiable wherever dfs(k) is, with
/// derivative value taylor_bt_term_deriv.
theorem taylor_bt_term_fn_has_derivative_at(dfs: Nat -> Real -> Real, k: Nat, b: Real, x0: Real) {
    is_derivative_fn(dfs(k), dfs(k.suc))
    implies has_derivative_at(taylor_bt_term_fn(dfs, b, k), x0,
        taylor_bt_term_deriv(dfs, b, x0, k))
} by {
    if is_derivative_fn(dfs(k), dfs(k.suc)) {
        is_derivative_fn_at(dfs(k), dfs(k.suc), x0)
        has_derivative_at(dfs(k), x0, dfs(k.suc, x0))
        derivative_pointwise_const_mul(from_nat[Real](k.factorial).inverse, dfs(k), x0,
            dfs(k.suc, x0))
        has_derivative_at(pointwise_mul(constant[Real, Real](from_nat[Real](k.factorial).inverse), dfs(k)),
            x0, from_nat[Real](k.factorial).inverse * dfs(k.suc, x0))
        taylor_shift_pow_has_derivative_at(b, k, x0)
        has_derivative_at(taylor_shift_pow(b, k), x0,
            -(from_nat[Real](k)) * (b - x0).pow(k - Nat.1))
        derivative_pointwise_mul(
            pointwise_mul(constant[Real, Real](from_nat[Real](k.factorial).inverse), dfs(k)),
            taylor_shift_pow(b, k), x0,
            from_nat[Real](k.factorial).inverse * dfs(k.suc, x0),
            -(from_nat[Real](k)) * (b - x0).pow(k - Nat.1))
        has_derivative_at(taylor_bt_term_fn(dfs, b, k), x0,
            pointwise_mul(constant[Real, Real](from_nat[Real](k.factorial).inverse), dfs(k), x0) *
                (-(from_nat[Real](k)) * (b - x0).pow(k - Nat.1)) +
            taylor_shift_pow(b, k, x0) *
                (from_nat[Real](k.factorial).inverse * dfs(k.suc, x0)))
        pointwise_mul(constant[Real, Real](from_nat[Real](k.factorial).inverse), dfs(k), x0) =
            constant[Real, Real](from_nat[Real](k.factorial).inverse, x0) * dfs(k, x0)
        constant[Real, Real](from_nat[Real](k.factorial).inverse, x0) =
            from_nat[Real](k.factorial).inverse
        pointwise_mul(constant[Real, Real](from_nat[Real](k.factorial).inverse), dfs(k), x0) =
            from_nat[Real](k.factorial).inverse * dfs(k, x0)
        taylor_shift_pow_value(b, k, x0)
        taylor_shift_pow(b, k, x0) = (b - x0).pow(k)
        real_mul_comm(from_nat[Real](k.factorial).inverse, dfs(k, x0))
        from_nat[Real](k.factorial).inverse * dfs(k, x0) =
            dfs(k, x0) * from_nat[Real](k.factorial).inverse
        dfs(k, x0) / from_nat[Real](k.factorial) =
            dfs(k, x0) * from_nat[Real](k.factorial).inverse
        from_nat[Real](k.factorial).inverse * dfs(k, x0) =
            dfs(k, x0) / from_nat[Real](k.factorial)
        pointwise_mul(constant[Real, Real](from_nat[Real](k.factorial).inverse), dfs(k), x0) =
            dfs(k, x0) / from_nat[Real](k.factorial)
        real_mul_comm(from_nat[Real](k.factorial).inverse, dfs(k.suc, x0))
        from_nat[Real](k.factorial).inverse * dfs(k.suc, x0) =
            dfs(k.suc, x0) * from_nat[Real](k.factorial).inverse
        dfs(k.suc, x0) / from_nat[Real](k.factorial) =
            dfs(k.suc, x0) * from_nat[Real](k.factorial).inverse
        from_nat[Real](k.factorial).inverse * dfs(k.suc, x0) =
            dfs(k.suc, x0) / from_nat[Real](k.factorial)
        pointwise_mul(constant[Real, Real](from_nat[Real](k.factorial).inverse), dfs(k), x0) *
            (-(from_nat[Real](k)) * (b - x0).pow(k - Nat.1)) +
        taylor_shift_pow(b, k, x0) *
            (from_nat[Real](k.factorial).inverse * dfs(k.suc, x0)) =
            (dfs(k, x0) / from_nat[Real](k.factorial)) *
                (-(from_nat[Real](k)) * (b - x0).pow(k - Nat.1)) +
            (b - x0).pow(k) * (dfs(k.suc, x0) / from_nat[Real](k.factorial))
        mul_neg_left(from_nat[Real](k), (b - x0).pow(k - Nat.1))
        -(from_nat[Real](k)) * (b - x0).pow(k - Nat.1) =
            -(from_nat[Real](k) * (b - x0).pow(k - Nat.1))
        (dfs(k, x0) / from_nat[Real](k.factorial)) *
            (-(from_nat[Real](k)) * (b - x0).pow(k - Nat.1)) =
            (dfs(k, x0) / from_nat[Real](k.factorial)) *
                (-(from_nat[Real](k) * (b - x0).pow(k - Nat.1)))
        mul_neg_right(dfs(k, x0) / from_nat[Real](k.factorial),
            from_nat[Real](k) * (b - x0).pow(k - Nat.1))
        (dfs(k, x0) / from_nat[Real](k.factorial)) *
            (-(from_nat[Real](k) * (b - x0).pow(k - Nat.1))) =
            -((dfs(k, x0) / from_nat[Real](k.factorial)) *
                (from_nat[Real](k) * (b - x0).pow(k - Nat.1)))
        mul_assoc(dfs(k, x0) / from_nat[Real](k.factorial), from_nat[Real](k),
            (b - x0).pow(k - Nat.1))
        ((dfs(k, x0) / from_nat[Real](k.factorial)) * from_nat[Real](k)) *
            (b - x0).pow(k - Nat.1) =
            (dfs(k, x0) / from_nat[Real](k.factorial)) *
                (from_nat[Real](k) * (b - x0).pow(k - Nat.1))
        (dfs(k, x0) / from_nat[Real](k.factorial)) *
            (-(from_nat[Real](k)) * (b - x0).pow(k - Nat.1)) =
            -(((dfs(k, x0) / from_nat[Real](k.factorial)) * from_nat[Real](k)) *
                (b - x0).pow(k - Nat.1))
        (dfs(k, x0) / from_nat[Real](k.factorial)) *
            (-(from_nat[Real](k)) * (b - x0).pow(k - Nat.1)) =
            -((dfs(k, x0) / from_nat[Real](k.factorial)) * from_nat[Real](k) *
                (b - x0).pow(k - Nat.1))
        real_mul_comm(b - x0, dfs(k.suc, x0) / from_nat[Real](k.factorial))
        (b - x0).pow(k) * (dfs(k.suc, x0) / from_nat[Real](k.factorial)) =
            (dfs(k.suc, x0) / from_nat[Real](k.factorial)) * (b - x0).pow(k)
        pointwise_mul(constant[Real, Real](from_nat[Real](k.factorial).inverse), dfs(k), x0) *
            (-(from_nat[Real](k)) * (b - x0).pow(k - Nat.1)) +
        taylor_shift_pow(b, k, x0) *
            (from_nat[Real](k.factorial).inverse * dfs(k.suc, x0)) =
            -((dfs(k, x0) / from_nat[Real](k.factorial)) * from_nat[Real](k) *
                (b - x0).pow(k - Nat.1)) +
            (b - x0).pow(k) * (dfs(k.suc, x0) / from_nat[Real](k.factorial))
        add_comm(-((dfs(k, x0) / from_nat[Real](k.factorial)) * from_nat[Real](k) *
                (b - x0).pow(k - Nat.1)),
            (b - x0).pow(k) * (dfs(k.suc, x0) / from_nat[Real](k.factorial)))
        -((dfs(k, x0) / from_nat[Real](k.factorial)) * from_nat[Real](k) *
                (b - x0).pow(k - Nat.1)) +
            (b - x0).pow(k) * (dfs(k.suc, x0) / from_nat[Real](k.factorial)) =
            (b - x0).pow(k) * (dfs(k.suc, x0) / from_nat[Real](k.factorial)) +
            -((dfs(k, x0) / from_nat[Real](k.factorial)) * from_nat[Real](k) *
                (b - x0).pow(k - Nat.1))
        (b - x0).pow(k) * (dfs(k.suc, x0) / from_nat[Real](k.factorial)) +
            -((dfs(k, x0) / from_nat[Real](k.factorial)) * from_nat[Real](k) *
                (b - x0).pow(k - Nat.1)) =
            (b - x0).pow(k) * (dfs(k.suc, x0) / from_nat[Real](k.factorial)) -
            (dfs(k, x0) / from_nat[Real](k.factorial)) * from_nat[Real](k) *
                (b - x0).pow(k - Nat.1)
        (b - x0).pow(k) * (dfs(k.suc, x0) / from_nat[Real](k.factorial)) -
            (dfs(k, x0) / from_nat[Real](k.factorial)) * from_nat[Real](k) *
                (b - x0).pow(k - Nat.1) =
            (dfs(k.suc, x0) / from_nat[Real](k.factorial)) * (b - x0).pow(k) -
            (dfs(k, x0) / from_nat[Real](k.factorial)) * from_nat[Real](k) *
                (b - x0).pow(k - Nat.1)
        pointwise_mul(constant[Real, Real](from_nat[Real](k.factorial).inverse), dfs(k), x0) *
            (-(from_nat[Real](k)) * (b - x0).pow(k - Nat.1)) +
        taylor_shift_pow(b, k, x0) *
            (from_nat[Real](k.factorial).inverse * dfs(k.suc, x0)) =
            (dfs(k.suc, x0) / from_nat[Real](k.factorial)) * (b - x0).pow(k) -
            (dfs(k, x0) / from_nat[Real](k.factorial)) * from_nat[Real](k) *
                (b - x0).pow(k - Nat.1)
        has_derivative_at(taylor_bt_term_fn(dfs, b, k), x0,
            (dfs(k.suc, x0) / from_nat[Real](k.factorial)) * (b - x0).pow(k) -
            (dfs(k, x0) / from_nat[Real](k.factorial)) * from_nat[Real](k) *
                (b - x0).pow(k - Nat.1))
        taylor_bt_term_deriv(dfs, b, x0, k) =
            (dfs(k.suc, x0) / from_nat[Real](k.factorial)) * (b - x0).pow(k) -
            (dfs(k, x0) / from_nat[Real](k.factorial)) * from_nat[Real](k) *
                (b - x0).pow(k - Nat.1)
        has_derivative_at(taylor_bt_term_fn(dfs, b, k), x0,
            taylor_bt_term_deriv(dfs, b, x0, k))
    }
}

/// The function x ↦ Σ_{k=0}^{m-1} taylor_bt_term(dfs, b, x, k).
define taylor_bt_sum_fn(dfs: Nat -> Real -> Real, b: Real, m: Nat) -> Real -> Real {
    function(x: Real) { partial(taylor_bt_term(dfs, b, x), m) }
}

/// The derivative value of the truncated backward Taylor sum at x.
define taylor_bt_sum_deriv(dfs: Nat -> Real -> Real, b: Real, x: Real, m: Nat) -> Real {
    partial(taylor_bt_term_deriv(dfs, b, x), m)
}

/// The zero-length backward Taylor sum is the constant zero function.
theorem taylor_bt_sum_fn_zero(dfs: Nat -> Real -> Real, b: Real) {
    taylor_bt_sum_fn(dfs, b, Nat.0) = constant[Real, Real](Real.0)
} by {
    forall(x: Real) {
        taylor_bt_sum_fn(dfs, b, Nat.0, x) = partial(taylor_bt_term(dfs, b, x), Nat.0)
        partial_zero(taylor_bt_term(dfs, b, x))
        partial(taylor_bt_term(dfs, b, x), Nat.0) = Real.0
        constant[Real, Real](Real.0, x) = Real.0
        taylor_bt_sum_fn(dfs, b, Nat.0, x) = constant[Real, Real](Real.0, x)
    }
    function_extensionality(taylor_bt_sum_fn(dfs, b, Nat.0), constant[Real, Real](Real.0))
    taylor_bt_sum_fn(dfs, b, Nat.0) = constant[Real, Real](Real.0)
}

/// The (m + 1)-term backward Taylor sum splits into the m-term sum plus the
/// m-th term.
theorem taylor_bt_sum_fn_suc(dfs: Nat -> Real -> Real, b: Real, m: Nat) {
    taylor_bt_sum_fn(dfs, b, m.suc) =
        pointwise_add(taylor_bt_sum_fn(dfs, b, m), taylor_bt_term_fn(dfs, b, m))
} by {
    forall(x: Real) {
        taylor_bt_sum_fn(dfs, b, m.suc, x) = partial(taylor_bt_term(dfs, b, x), m.suc)
        partial_split_last(taylor_bt_term(dfs, b, x), m)
        partial(taylor_bt_term(dfs, b, x), m.suc) =
            partial(taylor_bt_term(dfs, b, x), m) + taylor_bt_term(dfs, b, x, m)
        taylor_bt_sum_fn(dfs, b, m, x) = partial(taylor_bt_term(dfs, b, x), m)
        taylor_bt_term_fn_value(dfs, b, m, x)
        taylor_bt_term_fn(dfs, b, m, x) = taylor_bt_term(dfs, b, x, m)
        taylor_bt_sum_fn(dfs, b, m.suc, x) =
            taylor_bt_sum_fn(dfs, b, m, x) + taylor_bt_term_fn(dfs, b, m, x)
        pointwise_add(taylor_bt_sum_fn(dfs, b, m), taylor_bt_term_fn(dfs, b, m), x) =
            taylor_bt_sum_fn(dfs, b, m, x) + taylor_bt_term_fn(dfs, b, m, x)
        taylor_bt_sum_fn(dfs, b, m.suc, x) =
            pointwise_add(taylor_bt_sum_fn(dfs, b, m), taylor_bt_term_fn(dfs, b, m), x)
    }
    function_extensionality(taylor_bt_sum_fn(dfs, b, m.suc),
        pointwise_add(taylor_bt_sum_fn(dfs, b, m), taylor_bt_term_fn(dfs, b, m)))
    taylor_bt_sum_fn(dfs, b, m.suc) =
        pointwise_add(taylor_bt_sum_fn(dfs, b, m), taylor_bt_term_fn(dfs, b, m))
}

/// The derivative value of the (m + 1)-term sum splits off the m-th term.
theorem taylor_bt_sum_deriv_suc(dfs: Nat -> Real -> Real, b: Real, x: Real, m: Nat) {
    taylor_bt_sum_deriv(dfs, b, x, m.suc) =
        taylor_bt_sum_deriv(dfs, b, x, m) + taylor_bt_term_deriv(dfs, b, x, m)
} by {
    taylor_bt_sum_deriv(dfs, b, x, m.suc) = partial(taylor_bt_term_deriv(dfs, b, x), m.suc)
    partial_split_last(taylor_bt_term_deriv(dfs, b, x), m)
    partial(taylor_bt_term_deriv(dfs, b, x), m.suc) =
        partial(taylor_bt_term_deriv(dfs, b, x), m) + taylor_bt_term_deriv(dfs, b, x, m)
    taylor_bt_sum_deriv(dfs, b, x, m) = partial(taylor_bt_term_deriv(dfs, b, x), m)
    taylor_bt_sum_deriv(dfs, b, x, m.suc) =
        taylor_bt_sum_deriv(dfs, b, x, m) + taylor_bt_term_deriv(dfs, b, x, m)
}

/// The truncated backward Taylor sum of m terms is differentiable wherever each
/// of the first m derivative functions is differentiable, with derivative value
/// taylor_bt_sum_deriv.
theorem taylor_bt_sum_fn_has_derivative_at(dfs: Nat -> Real -> Real, b: Real, m: Nat, x0: Real) {
    forall(k: Nat) { k < m implies is_derivative_fn(dfs(k), dfs(k.suc)) }
    implies has_derivative_at(taylor_bt_sum_fn(dfs, b, m), x0,
        taylor_bt_sum_deriv(dfs, b, x0, m))
} by {
    define p(i: Nat) -> Bool {
        (forall(k: Nat) { k < i implies is_derivative_fn(dfs(k), dfs(k.suc)) })
        implies has_derivative_at(taylor_bt_sum_fn(dfs, b, i), x0,
            taylor_bt_sum_deriv(dfs, b, x0, i))
    }
    taylor_bt_sum_fn_zero(dfs, b)
    taylor_bt_sum_fn(dfs, b, Nat.0) = constant[Real, Real](Real.0)
    constant_has_derivative_at(Real.0, x0)
    has_derivative_at(constant[Real, Real](Real.0), x0, Real.0)
    function_eq_transport_predicate_rev(
        function(h: Real -> Real) { has_derivative_at(h, x0, Real.0) },
        taylor_bt_sum_fn(dfs, b, Nat.0), constant[Real, Real](Real.0))
    has_derivative_at(taylor_bt_sum_fn(dfs, b, Nat.0), x0, Real.0)
    taylor_bt_sum_deriv(dfs, b, x0, Nat.0) = partial(taylor_bt_term_deriv(dfs, b, x0), Nat.0)
    partial_zero(taylor_bt_term_deriv(dfs, b, x0))
    partial(taylor_bt_term_deriv(dfs, b, x0), Nat.0) = Real.0
    taylor_bt_sum_deriv(dfs, b, x0, Nat.0) = Real.0
    has_derivative_at(taylor_bt_sum_fn(dfs, b, Nat.0), x0,
        taylor_bt_sum_deriv(dfs, b, x0, Nat.0))
    p(Nat.0)
    forall(i: Nat) {
        if p(i) {
            if forall(k: Nat) { k < i.suc implies is_derivative_fn(dfs(k), dfs(k.suc)) } {
                forall(k: Nat) {
                    if k < i {
                        lt_imp_lt_suc(k, i)
                        k < i.suc
                        forall(k2: Nat) { k2 < i.suc implies is_derivative_fn(dfs(k2), dfs(k2.suc)) }
                        is_derivative_fn(dfs(k), dfs(k.suc))
                    }
                }
                forall(k: Nat) { k < i implies is_derivative_fn(dfs(k), dfs(k.suc)) }
                p(i) = ((forall(k: Nat) { k < i implies is_derivative_fn(dfs(k), dfs(k.suc)) })
                    implies has_derivative_at(taylor_bt_sum_fn(dfs, b, i), x0,
                        taylor_bt_sum_deriv(dfs, b, x0, i)))
                has_derivative_at(taylor_bt_sum_fn(dfs, b, i), x0,
                    taylor_bt_sum_deriv(dfs, b, x0, i))
                lt_suc(i)
                i < i.suc
                forall(k2: Nat) { k2 < i.suc implies is_derivative_fn(dfs(k2), dfs(k2.suc)) }
                is_derivative_fn(dfs(i), dfs(i.suc))
                taylor_bt_term_fn_has_derivative_at(dfs, i, b, x0)
                has_derivative_at(taylor_bt_term_fn(dfs, b, i), x0,
                    taylor_bt_term_deriv(dfs, b, x0, i))
                taylor_bt_sum_fn_suc(dfs, b, i)
                taylor_bt_sum_fn(dfs, b, i.suc) =
                    pointwise_add(taylor_bt_sum_fn(dfs, b, i), taylor_bt_term_fn(dfs, b, i))
                derivative_pointwise_add(taylor_bt_sum_fn(dfs, b, i),
                    taylor_bt_term_fn(dfs, b, i), x0,
                    taylor_bt_sum_deriv(dfs, b, x0, i),
                    taylor_bt_term_deriv(dfs, b, x0, i))
                has_derivative_at(pointwise_add(taylor_bt_sum_fn(dfs, b, i),
                        taylor_bt_term_fn(dfs, b, i)), x0,
                    taylor_bt_sum_deriv(dfs, b, x0, i) + taylor_bt_term_deriv(dfs, b, x0, i))
                function_eq_transport_predicate_rev(
                    function(h: Real -> Real) {
                        has_derivative_at(h, x0,
                            taylor_bt_sum_deriv(dfs, b, x0, i) + taylor_bt_term_deriv(dfs, b, x0, i))
                    },
                    taylor_bt_sum_fn(dfs, b, i.suc),
                    pointwise_add(taylor_bt_sum_fn(dfs, b, i), taylor_bt_term_fn(dfs, b, i)))
                has_derivative_at(taylor_bt_sum_fn(dfs, b, i.suc), x0,
                    taylor_bt_sum_deriv(dfs, b, x0, i) + taylor_bt_term_deriv(dfs, b, x0, i))
                taylor_bt_sum_deriv_suc(dfs, b, x0, i)
                taylor_bt_sum_deriv(dfs, b, x0, i.suc) =
                    taylor_bt_sum_deriv(dfs, b, x0, i) + taylor_bt_term_deriv(dfs, b, x0, i)
                has_derivative_at(taylor_bt_sum_fn(dfs, b, i.suc), x0,
                    taylor_bt_sum_deriv(dfs, b, x0, i.suc))
                p(i.suc)
            }
            p(i.suc)
        }
    }
    p(Nat.0) and forall(i: Nat) { p(i) implies p(i.suc) }
    alt_induction(p)
    forall(i: Nat) { p(i) }
    if forall(k: Nat) { k < m implies is_derivative_fn(dfs(k), dfs(k.suc)) } {
        p(m) = ((forall(k: Nat) { k < m implies is_derivative_fn(dfs(k), dfs(k.suc)) })
            implies has_derivative_at(taylor_bt_sum_fn(dfs, b, m), x0,
                taylor_bt_sum_deriv(dfs, b, x0, m)))
        has_derivative_at(taylor_bt_sum_fn(dfs, b, m), x0,
            taylor_bt_sum_deriv(dfs, b, x0, m))
    }
}

/// The image of the factorial of the successor in the reals factors.
theorem from_nat_factorial_suc_real(n: Nat) {
    from_nat[Real](n.suc.factorial) = from_nat[Real](n.suc) * from_nat[Real](n.factorial)
} by {
    factorial_suc_real(n)
    Real.from_rat(Rat.from_nat(n.suc.factorial)) =
        Real.from_rat(Rat.from_nat(n.suc)) * Real.from_rat(Rat.from_nat(n.factorial))
    from_nat_is_from_rat(n.suc.factorial)
    from_nat[Real](n.suc.factorial) = Real.from_rat(Rat.from_nat(n.suc.factorial))
    from_nat_is_from_rat(n.suc)
    from_nat[Real](n.suc) = Real.from_rat(Rat.from_nat(n.suc))
    from_nat_is_from_rat(n.factorial)
    from_nat[Real](n.factorial) = Real.from_rat(Rat.from_nat(n.factorial))
    from_nat[Real](n.suc.factorial) = from_nat[Real](n.suc) * from_nat[Real](n.factorial)
}

/// The image of a factorial in the reals is nonzero.
theorem taylor_factorial_real_ne_zero(n: Nat) {
    from_nat[Real](n.factorial) != Real.0
} by {
    factorial_pos(n)
    Real.from_rat(Rat.from_nat(n.factorial)) > Real.0
    from_nat_is_from_rat(n.factorial)
    from_nat[Real](n.factorial) = Real.from_rat(Rat.from_nat(n.factorial))
    from_nat[Real](n.factorial) > Real.0
    lt_imp_ne_symm(Real.0, from_nat[Real](n.factorial))
    from_nat[Real](n.factorial) != Real.0
}

/// The image of a successor in the reals is nonzero.
theorem from_nat_suc_ne_zero(n: Nat) {
    from_nat[Real](n.suc) != Real.0
} by {
    suc_pos(n)
    Real.from_rat(Rat.from_nat(n.suc)) > Real.0
    from_nat_is_from_rat(n.suc)
    from_nat[Real](n.suc) = Real.from_rat(Rat.from_nat(n.suc))
    from_nat[Real](n.suc) > Real.0
    lt_imp_ne_symm(Real.0, from_nat[Real](n.suc))
    from_nat[Real](n.suc) != Real.0
}

/// The zero-th power of zero is one, and the positive powers of zero are zero.
theorem taylor_zero_pow_suc(k: Nat) {
    Real.0.pow(k.suc) = Real.0
} by {
    semiring_zero_pow[Real](k.suc)
    alt_suc_ne_zero(k)
    k.suc != Nat.0
    Real.0.pow(k.suc) = Real.0
}

/// The zero-th backward Taylor term at x is the value of dfs(0) at x.
theorem taylor_bt_term_zero_at(dfs: Nat -> Real -> Real, b: Real, x: Real) {
    taylor_bt_term(dfs, b, x, Nat.0) = dfs(Nat.0, x)
} by {
    taylor_bt_term(dfs, b, x, Nat.0) =
        (dfs(Nat.0, x) / from_nat[Real](Nat.0.factorial)) * (b - x).pow(Nat.0)
    from_nat_factorial_zero_real
    from_nat[Real](Nat.0.factorial) = Real.1
    dfs(Nat.0, x) / from_nat[Real](Nat.0.factorial) = dfs(Nat.0, x) / Real.1
    real_div_one(dfs(Nat.0, x))
    dfs(Nat.0, x) / Real.1 = dfs(Nat.0, x)
    dfs(Nat.0, x) / from_nat[Real](Nat.0.factorial) = dfs(Nat.0, x)
    pow_zero[Real](b - x)
    (b - x).pow(Nat.0) = Real.1
    (dfs(Nat.0, x) / from_nat[Real](Nat.0.factorial)) * (b - x).pow(Nat.0) =
        dfs(Nat.0, x) * Real.1
    mul_one_right(dfs(Nat.0, x))
    dfs(Nat.0, x) * Real.1 = dfs(Nat.0, x)
    (dfs(Nat.0, x) / from_nat[Real](Nat.0.factorial)) * (b - x).pow(Nat.0) = dfs(Nat.0, x)
    taylor_bt_term(dfs, b, x, Nat.0) = dfs(Nat.0, x)
}

/// The positive-index backward Taylor terms vanish at b.
theorem taylor_bt_term_zero_shift(dfs: Nat -> Real -> Real, b: Real, k: Nat) {
    taylor_bt_term(dfs, b, b, k.suc) = Real.0
} by {
    taylor_bt_term(dfs, b, b, k.suc) =
        (dfs(k.suc, b) / from_nat[Real](k.suc.factorial)) * (b - b).pow(k.suc)
    b - b = Real.0
    (b - b).pow(k.suc) = Real.0.pow(k.suc)
    taylor_zero_pow_suc(k)
    Real.0.pow(k.suc) = Real.0
    (b - b).pow(k.suc) = Real.0
    mul_zero_right(dfs(k.suc, b) / from_nat[Real](k.suc.factorial))
    (dfs(k.suc, b) / from_nat[Real](k.suc.factorial)) * Real.0 = Real.0
    (dfs(k.suc, b) / from_nat[Real](k.suc.factorial)) * (b - b).pow(k.suc) = Real.0
    taylor_bt_term(dfs, b, b, k.suc) = Real.0
}

/// A partial sum whose terms are all zero is zero.
theorem taylor_partial_zero_on(f: Nat -> Real, n: Nat) {
    forall(k: Nat) { k < n implies f(k) = Real.0 }
    implies partial(f, n) = Real.0
} by {
    define p(i: Nat) -> Bool {
        (forall(k: Nat) { k < i implies f(k) = Real.0 })
        implies partial(f, i) = Real.0
    }
    partial_zero(f)
    partial(f, Nat.0) = Real.0
    p(Nat.0)
    forall(i: Nat) {
        if p(i) {
            if forall(k: Nat) { k < i.suc implies f(k) = Real.0 } {
                forall(k: Nat) {
                    if k < i {
                        lt_imp_lt_suc(k, i)
                        k < i.suc
                        forall(k2: Nat) { k2 < i.suc implies f(k2) = Real.0 }
                        f(k) = Real.0
                    }
                }
                forall(k: Nat) { k < i implies f(k) = Real.0 }
                p(i) = ((forall(k: Nat) { k < i implies f(k) = Real.0 })
                    implies partial(f, i) = Real.0)
                partial(f, i) = Real.0
                lt_suc(i)
                i < i.suc
                forall(k2: Nat) { k2 < i.suc implies f(k2) = Real.0 }
                f(i) = Real.0
                partial_split_last(f, i)
                partial(f, i.suc) = partial(f, i) + f(i)
                add_zero_right(partial(f, i))
                partial(f, i) + Real.0 = partial(f, i)
                partial(f, i) + f(i) = partial(f, i)
                partial(f, i.suc) = partial(f, i)
                partial(f, i.suc) = Real.0
                p(i.suc)
            }
            p(i.suc)
        }
    }
    p(Nat.0) and forall(i: Nat) { p(i) implies p(i.suc) }
    alt_induction(p)
    forall(i: Nat) { p(i) }
    if forall(k: Nat) { k < n implies f(k) = Real.0 } {
        p(n) = ((forall(k: Nat) { k < n implies f(k) = Real.0 })
            implies partial(f, n) = Real.0)
        partial(f, n) = Real.0
    }
}

/// The real image of (n + 1)! is the real image of n + 1 times the real image
/// of n!, so dividing by the latter factors out the former.
theorem taylor_ratio_mul(n: Nat, a: Real) {
    (a / from_nat[Real](n.suc.factorial)) * from_nat[Real](n.suc) =
        a / from_nat[Real](n.factorial)
} by {
    a / from_nat[Real](n.suc.factorial) = a * from_nat[Real](n.suc.factorial).inverse
    from_nat_factorial_suc_real(n)
    from_nat[Real](n.suc.factorial) = from_nat[Real](n.suc) * from_nat[Real](n.factorial)
    from_nat[Real](n.suc.factorial).inverse =
        (from_nat[Real](n.suc) * from_nat[Real](n.factorial)).inverse
    inverse_dist(from_nat[Real](n.suc), from_nat[Real](n.factorial))
    (from_nat[Real](n.suc) * from_nat[Real](n.factorial)).inverse =
        from_nat[Real](n.factorial).inverse * from_nat[Real](n.suc).inverse
    from_nat[Real](n.suc.factorial).inverse =
        from_nat[Real](n.factorial).inverse * from_nat[Real](n.suc).inverse
    a * from_nat[Real](n.suc.factorial).inverse =
        a * (from_nat[Real](n.factorial).inverse * from_nat[Real](n.suc).inverse)
    mul_assoc(a, from_nat[Real](n.factorial).inverse, from_nat[Real](n.suc).inverse)
    a * (from_nat[Real](n.factorial).inverse * from_nat[Real](n.suc).inverse) =
        (a * from_nat[Real](n.factorial).inverse) * from_nat[Real](n.suc).inverse
    a * from_nat[Real](n.suc.factorial).inverse =
        (a * from_nat[Real](n.factorial).inverse) * from_nat[Real](n.suc).inverse
    (a / from_nat[Real](n.suc.factorial)) * from_nat[Real](n.suc) =
        ((a * from_nat[Real](n.factorial).inverse) * from_nat[Real](n.suc).inverse) *
            from_nat[Real](n.suc)
    mul_assoc(a * from_nat[Real](n.factorial).inverse,
        from_nat[Real](n.suc).inverse, from_nat[Real](n.suc))
    ((a * from_nat[Real](n.factorial).inverse) * from_nat[Real](n.suc).inverse) *
        from_nat[Real](n.suc) =
        (a * from_nat[Real](n.factorial).inverse) *
            (from_nat[Real](n.suc).inverse * from_nat[Real](n.suc))
    real_mul_comm(from_nat[Real](n.suc).inverse, from_nat[Real](n.suc))
    from_nat[Real](n.suc).inverse * from_nat[Real](n.suc) =
        from_nat[Real](n.suc) * from_nat[Real](n.suc).inverse
    from_nat_suc_ne_zero(n)
    mul_inverse(from_nat[Real](n.suc))
    from_nat[Real](n.suc) * from_nat[Real](n.suc).inverse = Real.1
    from_nat[Real](n.suc).inverse * from_nat[Real](n.suc) = Real.1
    (a * from_nat[Real](n.factorial).inverse) *
        (from_nat[Real](n.suc).inverse * from_nat[Real](n.suc)) =
        (a * from_nat[Real](n.factorial).inverse) * Real.1
    mul_one_right(a * from_nat[Real](n.factorial).inverse)
    (a * from_nat[Real](n.factorial).inverse) * Real.1 =
        a * from_nat[Real](n.factorial).inverse
    (a * from_nat[Real](n.factorial).inverse) *
        (from_nat[Real](n.suc).inverse * from_nat[Real](n.suc)) =
        a * from_nat[Real](n.factorial).inverse
    ((a * from_nat[Real](n.factorial).inverse) * from_nat[Real](n.suc).inverse) *
        from_nat[Real](n.suc) = a * from_nat[Real](n.factorial).inverse
    (a / from_nat[Real](n.suc.factorial)) * from_nat[Real](n.suc) =
        a * from_nat[Real](n.factorial).inverse
    a / from_nat[Real](n.factorial) = a * from_nat[Real](n.factorial).inverse
    (a / from_nat[Real](n.suc.factorial)) * from_nat[Real](n.suc) =
        a / from_nat[Real](n.factorial)
}

/// Dividing by n! and then by n + 1 is dividing by (n + 1)!.
theorem taylor_ratio_div(n: Nat, a: Real) {
    (a / from_nat[Real](n.factorial)) / from_nat[Real](n.suc) =
        a / from_nat[Real](n.suc.factorial)
} by {
    (a / from_nat[Real](n.factorial)) / from_nat[Real](n.suc) =
        (a / from_nat[Real](n.factorial)) * from_nat[Real](n.suc).inverse
    a / from_nat[Real](n.factorial) = a * from_nat[Real](n.factorial).inverse
    (a / from_nat[Real](n.factorial)) * from_nat[Real](n.suc).inverse =
        (a * from_nat[Real](n.factorial).inverse) * from_nat[Real](n.suc).inverse
    mul_assoc(a, from_nat[Real](n.factorial).inverse, from_nat[Real](n.suc).inverse)
    (a * from_nat[Real](n.factorial).inverse) * from_nat[Real](n.suc).inverse =
        a * (from_nat[Real](n.factorial).inverse * from_nat[Real](n.suc).inverse)
    inverse_dist(from_nat[Real](n.suc), from_nat[Real](n.factorial))
    (from_nat[Real](n.suc) * from_nat[Real](n.factorial)).inverse =
        from_nat[Real](n.factorial).inverse * from_nat[Real](n.suc).inverse
    real_mul_comm(from_nat[Real](n.suc), from_nat[Real](n.factorial))
    from_nat[Real](n.suc) * from_nat[Real](n.factorial) =
        from_nat[Real](n.factorial) * from_nat[Real](n.suc)
    (from_nat[Real](n.factorial) * from_nat[Real](n.suc)).inverse =
        from_nat[Real](n.factorial).inverse * from_nat[Real](n.suc).inverse
    from_nat_factorial_suc_real(n)
    from_nat[Real](n.suc.factorial) = from_nat[Real](n.suc) * from_nat[Real](n.factorial)
    from_nat[Real](n.suc.factorial) = from_nat[Real](n.factorial) * from_nat[Real](n.suc)
    from_nat[Real](n.suc.factorial).inverse =
        from_nat[Real](n.factorial).inverse * from_nat[Real](n.suc).inverse
    a * (from_nat[Real](n.factorial).inverse * from_nat[Real](n.suc).inverse) =
        a * from_nat[Real](n.suc.factorial).inverse
    (a * from_nat[Real](n.factorial).inverse) * from_nat[Real](n.suc).inverse =
        a * from_nat[Real](n.suc.factorial).inverse
    (a / from_nat[Real](n.factorial)) * from_nat[Real](n.suc).inverse =
        a * from_nat[Real](n.suc.factorial).inverse
    (a / from_nat[Real](n.factorial)) / from_nat[Real](n.suc) =
        a * from_nat[Real](n.suc.factorial).inverse
    a / from_nat[Real](n.suc.factorial) = a * from_nat[Real](n.suc.factorial).inverse
    (a / from_nat[Real](n.factorial)) / from_nat[Real](n.suc) =
        a / from_nat[Real](n.suc.factorial)
}

/// The Lagrange remainder coefficient of order n: the value of the truncated
/// backward Taylor remainder at a divided by (b - a)^(n + 1).
define taylor_aux_coeff(f: Real -> Real, dfs: Nat -> Real -> Real, a: Real, b: Real, n: Nat) -> Real {
    (f(b) - partial(taylor_bt_term(dfs, b, a), n.suc)) / (b - a).pow(n.suc)
}

/// The value of the auxiliary function of the n-th order Taylor theorem at x.
define taylor_aux_val(f: Real -> Real, dfs: Nat -> Real -> Real, a: Real, b: Real, n: Nat, x: Real) -> Real {
    f(b) - partial(taylor_bt_term(dfs, b, x), n.suc) -
        taylor_aux_coeff(f, dfs, a, b, n) * (b - x).pow(n.suc)
}

/// The auxiliary function of the n-th order Taylor theorem, as a pointwise
/// combination of constants, the truncated backward Taylor sum, and the
/// shifted power (b - x)^(n + 1).
define taylor_aux_fn(f: Real -> Real, dfs: Nat -> Real -> Real, a: Real, b: Real, n: Nat) -> Real -> Real {
    pointwise_add(
        pointwise_add(
            constant[Real, Real](f(b)),
            pointwise_neg(taylor_bt_sum_fn(dfs, b, n.suc))),
        pointwise_neg(pointwise_mul(
            constant[Real, Real](taylor_aux_coeff(f, dfs, a, b, n)),
            taylor_shift_pow(b, n.suc))))
}

/// The derivative value of the auxiliary function of the n-th order Taylor
/// theorem at x: minus the derivative of the truncated sum plus (n + 1) times
/// the remainder coefficient times (b - x)^n.
define taylor_aux_deriv_val(f: Real -> Real, dfs: Nat -> Real -> Real, a: Real, b: Real, n: Nat, x: Real) -> Real {
    -partial(taylor_bt_term_deriv(dfs, b, x), n.suc) +
        from_nat[Real](n.suc) * taylor_aux_coeff(f, dfs, a, b, n) * (b - x).pow(n)
}

/// The derivative function of the auxiliary function of the n-th order Taylor
/// theorem.
define taylor_aux_deriv_fn(f: Real -> Real, dfs: Nat -> Real -> Real, a: Real, b: Real, n: Nat) -> Real -> Real {
    function(x: Real) { taylor_aux_deriv_val(f, dfs, a, b, n, x) }
}

/// The value of the auxiliary function at x is the backward remainder minus
/// the remainder term.
theorem taylor_aux_fn_value(f: Real -> Real, dfs: Nat -> Real -> Real, a: Real, b: Real, n: Nat, x: Real) {
    taylor_aux_fn(f, dfs, a, b, n, x) = taylor_aux_val(f, dfs, a, b, n, x)
} by {
    taylor_aux_fn(f, dfs, a, b, n, x) = pointwise_add(
        pointwise_add(
            constant[Real, Real](f(b)),
            pointwise_neg(taylor_bt_sum_fn(dfs, b, n.suc))),
        pointwise_neg(pointwise_mul(
            constant[Real, Real](taylor_aux_coeff(f, dfs, a, b, n)),
            taylor_shift_pow(b, n.suc))), x)
    pointwise_add(pointwise_add(
            constant[Real, Real](f(b)),
            pointwise_neg(taylor_bt_sum_fn(dfs, b, n.suc))),
        pointwise_neg(pointwise_mul(
            constant[Real, Real](taylor_aux_coeff(f, dfs, a, b, n)),
            taylor_shift_pow(b, n.suc))), x) =
        pointwise_add(constant[Real, Real](f(b)),
            pointwise_neg(taylor_bt_sum_fn(dfs, b, n.suc)), x) +
        pointwise_neg(pointwise_mul(
            constant[Real, Real](taylor_aux_coeff(f, dfs, a, b, n)),
            taylor_shift_pow(b, n.suc)), x)
    pointwise_add(constant[Real, Real](f(b)),
        pointwise_neg(taylor_bt_sum_fn(dfs, b, n.suc)), x) =
        constant[Real, Real](f(b), x) + pointwise_neg(taylor_bt_sum_fn(dfs, b, n.suc), x)
    constant[Real, Real](f(b), x) = f(b)
    pointwise_neg(taylor_bt_sum_fn(dfs, b, n.suc), x) = -taylor_bt_sum_fn(dfs, b, n.suc, x)
    taylor_bt_sum_fn(dfs, b, n.suc, x) = partial(taylor_bt_term(dfs, b, x), n.suc)
    pointwise_neg(taylor_bt_sum_fn(dfs, b, n.suc), x) = -partial(taylor_bt_term(dfs, b, x), n.suc)
    pointwise_add(constant[Real, Real](f(b)),
        pointwise_neg(taylor_bt_sum_fn(dfs, b, n.suc)), x) =
        f(b) + -partial(taylor_bt_term(dfs, b, x), n.suc)
    f(b) + -partial(taylor_bt_term(dfs, b, x), n.suc) =
        f(b) - partial(taylor_bt_term(dfs, b, x), n.suc)
    pointwise_add(constant[Real, Real](f(b)),
        pointwise_neg(taylor_bt_sum_fn(dfs, b, n.suc)), x) =
        f(b) - partial(taylor_bt_term(dfs, b, x), n.suc)
    pointwise_mul(constant[Real, Real](taylor_aux_coeff(f, dfs, a, b, n)),
        taylor_shift_pow(b, n.suc), x) =
        constant[Real, Real](taylor_aux_coeff(f, dfs, a, b, n), x) *
            taylor_shift_pow(b, n.suc, x)
    constant[Real, Real](taylor_aux_coeff(f, dfs, a, b, n), x) =
        taylor_aux_coeff(f, dfs, a, b, n)
    taylor_shift_pow_value(b, n.suc, x)
    taylor_shift_pow(b, n.suc, x) = (b - x).pow(n.suc)
    pointwise_mul(constant[Real, Real](taylor_aux_coeff(f, dfs, a, b, n)),
        taylor_shift_pow(b, n.suc), x) =
        taylor_aux_coeff(f, dfs, a, b, n) * (b - x).pow(n.suc)
    pointwise_neg(pointwise_mul(
        constant[Real, Real](taylor_aux_coeff(f, dfs, a, b, n)),
        taylor_shift_pow(b, n.suc)), x) =
        -(pointwise_mul(constant[Real, Real](taylor_aux_coeff(f, dfs, a, b, n)),
            taylor_shift_pow(b, n.suc), x))
    pointwise_neg(pointwise_mul(
        constant[Real, Real](taylor_aux_coeff(f, dfs, a, b, n)),
        taylor_shift_pow(b, n.suc)), x) =
        -(taylor_aux_coeff(f, dfs, a, b, n) * (b - x).pow(n.suc))
    pointwise_add(pointwise_add(
            constant[Real, Real](f(b)),
            pointwise_neg(taylor_bt_sum_fn(dfs, b, n.suc))),
        pointwise_neg(pointwise_mul(
            constant[Real, Real](taylor_aux_coeff(f, dfs, a, b, n)),
            taylor_shift_pow(b, n.suc))), x) =
        (f(b) - partial(taylor_bt_term(dfs, b, x), n.suc)) +
            -(taylor_aux_coeff(f, dfs, a, b, n) * (b - x).pow(n.suc))
    (f(b) - partial(taylor_bt_term(dfs, b, x), n.suc)) +
        -(taylor_aux_coeff(f, dfs, a, b, n) * (b - x).pow(n.suc)) =
        f(b) - partial(taylor_bt_term(dfs, b, x), n.suc) -
            taylor_aux_coeff(f, dfs, a, b, n) * (b - x).pow(n.suc)
    pointwise_add(pointwise_add(
            constant[Real, Real](f(b)),
            pointwise_neg(taylor_bt_sum_fn(dfs, b, n.suc))),
        pointwise_neg(pointwise_mul(
            constant[Real, Real](taylor_aux_coeff(f, dfs, a, b, n)),
            taylor_shift_pow(b, n.suc))), x) =
        f(b) - partial(taylor_bt_term(dfs, b, x), n.suc) -
            taylor_aux_coeff(f, dfs, a, b, n) * (b - x).pow(n.suc)
    taylor_aux_fn(f, dfs, a, b, n, x) =
        f(b) - partial(taylor_bt_term(dfs, b, x), n.suc) -
            taylor_aux_coeff(f, dfs, a, b, n) * (b - x).pow(n.suc)
    taylor_aux_val(f, dfs, a, b, n, x) =
        f(b) - partial(taylor_bt_term(dfs, b, x), n.suc) -
            taylor_aux_coeff(f, dfs, a, b, n) * (b - x).pow(n.suc)
    taylor_aux_fn(f, dfs, a, b, n, x) = taylor_aux_val(f, dfs, a, b, n, x)
}

/// The auxiliary function of the n-th order Taylor theorem vanishes at a.
theorem taylor_aux_at_a_zero(f: Real -> Real, dfs: Nat -> Real -> Real, a: Real, b: Real, n: Nat) {
    a < b implies taylor_aux_val(f, dfs, a, b, n, a) = Real.0
} by {
    if a < b {
        taylor_aux_val(f, dfs, a, b, n, a) = f(b) - partial(taylor_bt_term(dfs, b, a), n.suc) -
            taylor_aux_coeff(f, dfs, a, b, n) * (b - a).pow(n.suc)
        taylor_aux_coeff(f, dfs, a, b, n) =
            (f(b) - partial(taylor_bt_term(dfs, b, a), n.suc)) / (b - a).pow(n.suc)
        lt_imp_ne_symm(a, b)
        b != a
        sub_ne_zero_of_ne(b, a)
        b - a != Real.0
        pow_not_zero[Real](b - a, n.suc)
        (b - a).pow(n.suc) != Real.0
        div_mul_cancel_denominator(f(b) - partial(taylor_bt_term(dfs, b, a), n.suc),
            (b - a).pow(n.suc))
        ((f(b) - partial(taylor_bt_term(dfs, b, a), n.suc)) / (b - a).pow(n.suc)) *
            (b - a).pow(n.suc) = f(b) - partial(taylor_bt_term(dfs, b, a), n.suc)
        taylor_aux_coeff(f, dfs, a, b, n) * (b - a).pow(n.suc) =
            f(b) - partial(taylor_bt_term(dfs, b, a), n.suc)
        taylor_aux_val(f, dfs, a, b, n, a) =
            f(b) - partial(taylor_bt_term(dfs, b, a), n.suc) -
                (f(b) - partial(taylor_bt_term(dfs, b, a), n.suc))
        (f(b) - partial(taylor_bt_term(dfs, b, a), n.suc)) -
            (f(b) - partial(taylor_bt_term(dfs, b, a), n.suc)) = Real.0
        f(b) - partial(taylor_bt_term(dfs, b, a), n.suc) -
            (f(b) - partial(taylor_bt_term(dfs, b, a), n.suc)) = Real.0
        taylor_aux_val(f, dfs, a, b, n, a) = Real.0
    }
}

/// The auxiliary function of the n-th order Taylor theorem vanishes at b.
theorem taylor_aux_at_b_zero(f: Real -> Real, dfs: Nat -> Real -> Real, a: Real, b: Real, n: Nat) {
    dfs(Nat.0) = f
    implies taylor_aux_val(f, dfs, a, b, n, b) = Real.0
} by {
    if dfs(Nat.0) = f {
        taylor_aux_val(f, dfs, a, b, n, b) = f(b) - partial(taylor_bt_term(dfs, b, b), n.suc) -
            taylor_aux_coeff(f, dfs, a, b, n) * (b - b).pow(n.suc)
        b - b = Real.0
        (b - b).pow(n.suc) = Real.0.pow(n.suc)
        taylor_zero_pow_suc(n)
        Real.0.pow(n.suc) = Real.0
        (b - b).pow(n.suc) = Real.0
        mul_zero_right(taylor_aux_coeff(f, dfs, a, b, n))
        taylor_aux_coeff(f, dfs, a, b, n) * Real.0 = Real.0
        taylor_aux_coeff(f, dfs, a, b, n) * (b - b).pow(n.suc) = Real.0
        taylor_aux_val(f, dfs, a, b, n, b) =
            f(b) - partial(taylor_bt_term(dfs, b, b), n.suc) - Real.0
        partial_shift_suc(taylor_bt_term(dfs, b, b), n)
        taylor_bt_term(dfs, b, b, Nat.0) + partial(compose(taylor_bt_term(dfs, b, b), Nat.suc), n) =
            partial(taylor_bt_term(dfs, b, b), n.suc)
        taylor_bt_term_zero_at(dfs, b, b)
        taylor_bt_term(dfs, b, b, Nat.0) = dfs(Nat.0, b)
        dfs(Nat.0, b) = f(b)
        taylor_bt_term(dfs, b, b, Nat.0) = f(b)
        forall(k: Nat) {
            if k < n {
                taylor_bt_term_zero_shift(dfs, b, k)
                taylor_bt_term(dfs, b, b, k.suc) = Real.0
                compose(taylor_bt_term(dfs, b, b), Nat.suc, k) =
                    taylor_bt_term(dfs, b, b, Nat.suc(k))
                Nat.suc(k) = k.suc
                compose(taylor_bt_term(dfs, b, b), Nat.suc, k) =
                    taylor_bt_term(dfs, b, b, k.suc)
                compose(taylor_bt_term(dfs, b, b), Nat.suc, k) = Real.0
            }
        }
        forall(k: Nat) { k < n implies compose(taylor_bt_term(dfs, b, b), Nat.suc, k) = Real.0 }
        taylor_partial_zero_on(compose(taylor_bt_term(dfs, b, b), Nat.suc), n)
        partial(compose(taylor_bt_term(dfs, b, b), Nat.suc), n) = Real.0
        taylor_bt_term(dfs, b, b, Nat.0) + partial(compose(taylor_bt_term(dfs, b, b), Nat.suc), n) =
            f(b) + Real.0
        add_zero_right(f(b))
        f(b) + Real.0 = f(b)
        taylor_bt_term(dfs, b, b, Nat.0) + partial(compose(taylor_bt_term(dfs, b, b), Nat.suc), n) =
            f(b)
        partial(taylor_bt_term(dfs, b, b), n.suc) = f(b)
        f(b) - partial(taylor_bt_term(dfs, b, b), n.suc) - Real.0 = f(b) - f(b) - Real.0
        f(b) - f(b) = Real.0
        Real.0 - Real.0 = Real.0
        f(b) - f(b) - Real.0 = Real.0
        f(b) - partial(taylor_bt_term(dfs, b, b), n.suc) - Real.0 = Real.0
        taylor_aux_val(f, dfs, a, b, n, b) = Real.0
    }
}

/// The auxiliary function of the n-th order Taylor theorem takes equal values
/// at the endpoints.
theorem taylor_aux_endpoints_equal(f: Real -> Real, dfs: Nat -> Real -> Real, a: Real, b: Real, n: Nat) {
    a < b and dfs(Nat.0) = f
    implies taylor_aux_fn(f, dfs, a, b, n, a) = taylor_aux_fn(f, dfs, a, b, n, b)
} by {
    if a < b and dfs(Nat.0) = f {
        taylor_aux_fn_value(f, dfs, a, b, n, a)
        taylor_aux_fn(f, dfs, a, b, n, a) = taylor_aux_val(f, dfs, a, b, n, a)
        taylor_aux_at_a_zero(f, dfs, a, b, n)
        taylor_aux_val(f, dfs, a, b, n, a) = Real.0
        taylor_aux_fn(f, dfs, a, b, n, a) = Real.0
        taylor_aux_fn_value(f, dfs, a, b, n, b)
        taylor_aux_fn(f, dfs, a, b, n, b) = taylor_aux_val(f, dfs, a, b, n, b)
        taylor_aux_at_b_zero(f, dfs, a, b, n)
        taylor_aux_val(f, dfs, a, b, n, b) = Real.0
        taylor_aux_fn(f, dfs, a, b, n, b) = Real.0
        taylor_aux_fn(f, dfs, a, b, n, a) = taylor_aux_fn(f, dfs, a, b, n, b)
    }
}

/// A global derivative function makes the function continuous on every closed
/// interval.
theorem is_derivative_fn_imp_continuous_on_closed(f: Real -> Real, df: Real -> Real, u: Real, v: Real) {
    is_derivative_fn(f, df) implies continuous_on_closed(f, u, v)
} by {
    if is_derivative_fn(f, df) {
        forall(x: Real) {
            if closed_interval_set(u, v).contains(x) {
                is_derivative_fn_imp_continuous_at(f, df, x)
                continuous_at(f, x)
            }
        }
        continuous_on_closed(f, u, v) = forall(x: Real) {
            closed_interval_set(u, v).contains(x) implies continuous_at(f, x)
        }
        continuous_on_closed(f, u, v)
    }
}

/// The first entry of a derivative chain is f.
theorem taylor_chain_first(f: Real -> Real, dfs: Nat -> Real -> Real, n: Nat) {
    is_derivative_chain(f, dfs, n) implies dfs(Nat.0) = f
} by {
    if is_derivative_chain(f, dfs, n) {
        is_derivative_chain(f, dfs, n) = (dfs(Nat.0) = f and forall(k: Nat) {
            k < n implies is_derivative_fn(dfs(k), dfs(k.suc))
        })
        dfs(Nat.0) = f
    }
}

/// Every entry of a derivative chain is a derivative function of its
/// predecessor.
theorem taylor_chain_all(f: Real -> Real, dfs: Nat -> Real -> Real, n: Nat, k: Nat) {
    is_derivative_chain(f, dfs, n) and k < n implies is_derivative_fn(dfs(k), dfs(k.suc))
} by {
    if is_derivative_chain(f, dfs, n) and k < n {
        is_derivative_chain(f, dfs, n) = (dfs(Nat.0) = f and forall(k2: Nat) {
            k2 < n implies is_derivative_fn(dfs(k2), dfs(k2.suc))
        })
        forall(k2: Nat) {
            k2 < n implies is_derivative_fn(dfs(k2), dfs(k2.suc))
        }
        k < n
        is_derivative_fn(dfs(k), dfs(k.suc))
    }
}

/// The auxiliary function of the n-th order Taylor theorem is differentiable
/// everywhere whenever dfs is a derivative chain of length n + 1, with
/// derivative function taylor_aux_deriv_fn.
theorem taylor_aux_fn_is_derivative_fn(f: Real -> Real, dfs: Nat -> Real -> Real, a: Real, b: Real, n: Nat) {
    is_derivative_chain(f, dfs, n.suc)
    implies is_derivative_fn(taylor_aux_fn(f, dfs, a, b, n), taylor_aux_deriv_fn(f, dfs, a, b, n))
} by {
    if is_derivative_chain(f, dfs, n.suc) {
        forall(x0: Real) {
            forall(k: Nat) {
                if k < n.suc {
                    taylor_chain_all(f, dfs, n.suc, k)
                    is_derivative_chain(f, dfs, n.suc) and k < n.suc implies is_derivative_fn(dfs(k), dfs(k.suc))
                    k < n.suc
                    is_derivative_fn(dfs(k), dfs(k.suc))
                }
            }
            forall(k: Nat) {
                k < n.suc implies is_derivative_fn(dfs(k), dfs(k.suc))
            }
            taylor_bt_sum_fn_has_derivative_at(dfs, b, n.suc, x0)
            has_derivative_at(taylor_bt_sum_fn(dfs, b, n.suc), x0,
                taylor_bt_sum_deriv(dfs, b, x0, n.suc))
            derivative_pointwise_neg(taylor_bt_sum_fn(dfs, b, n.suc), x0,
                taylor_bt_sum_deriv(dfs, b, x0, n.suc))
            has_derivative_at(pointwise_neg(taylor_bt_sum_fn(dfs, b, n.suc)), x0,
                -(taylor_bt_sum_deriv(dfs, b, x0, n.suc)))
            constant_has_derivative_at(f(b), x0)
            has_derivative_at(constant[Real, Real](f(b)), x0, Real.0)
            derivative_pointwise_add(constant[Real, Real](f(b)),
                pointwise_neg(taylor_bt_sum_fn(dfs, b, n.suc)), x0, Real.0,
                -(taylor_bt_sum_deriv(dfs, b, x0, n.suc)))
            has_derivative_at(pointwise_add(constant[Real, Real](f(b)),
                    pointwise_neg(taylor_bt_sum_fn(dfs, b, n.suc))), x0,
                Real.0 + -(taylor_bt_sum_deriv(dfs, b, x0, n.suc)))
            taylor_shift_pow_suc_has_derivative_at(b, n, x0)
            has_derivative_at(taylor_shift_pow(b, n.suc), x0,
                -(from_nat[Real](n.suc)) * (b - x0).pow(n))
            derivative_pointwise_const_mul(taylor_aux_coeff(f, dfs, a, b, n),
                taylor_shift_pow(b, n.suc), x0,
                -(from_nat[Real](n.suc)) * (b - x0).pow(n))
            has_derivative_at(pointwise_mul(constant[Real, Real](taylor_aux_coeff(f, dfs, a, b, n)),
                    taylor_shift_pow(b, n.suc)), x0,
                taylor_aux_coeff(f, dfs, a, b, n) *
                    (-(from_nat[Real](n.suc)) * (b - x0).pow(n)))
            derivative_pointwise_neg(pointwise_mul(constant[Real, Real](taylor_aux_coeff(f, dfs, a, b, n)),
                taylor_shift_pow(b, n.suc)), x0,
                taylor_aux_coeff(f, dfs, a, b, n) *
                    (-(from_nat[Real](n.suc)) * (b - x0).pow(n)))
            has_derivative_at(pointwise_neg(pointwise_mul(constant[Real, Real](
                        taylor_aux_coeff(f, dfs, a, b, n)),
                    taylor_shift_pow(b, n.suc))), x0,
                -(taylor_aux_coeff(f, dfs, a, b, n) *
                    (-(from_nat[Real](n.suc)) * (b - x0).pow(n))))
            mul_neg_left(from_nat[Real](n.suc), (b - x0).pow(n))
            -(from_nat[Real](n.suc)) * (b - x0).pow(n) =
                -(from_nat[Real](n.suc) * (b - x0).pow(n))
            taylor_aux_coeff(f, dfs, a, b, n) *
                (-(from_nat[Real](n.suc)) * (b - x0).pow(n)) =
                taylor_aux_coeff(f, dfs, a, b, n) *
                    (-(from_nat[Real](n.suc) * (b - x0).pow(n)))
            mul_neg_right(taylor_aux_coeff(f, dfs, a, b, n),
                from_nat[Real](n.suc) * (b - x0).pow(n))
            taylor_aux_coeff(f, dfs, a, b, n) *
                (-(from_nat[Real](n.suc) * (b - x0).pow(n))) =
                -(taylor_aux_coeff(f, dfs, a, b, n) *
                    (from_nat[Real](n.suc) * (b - x0).pow(n)))
            taylor_aux_coeff(f, dfs, a, b, n) *
                (-(from_nat[Real](n.suc)) * (b - x0).pow(n)) =
                -(taylor_aux_coeff(f, dfs, a, b, n) *
                    (from_nat[Real](n.suc) * (b - x0).pow(n)))
            mul_assoc(taylor_aux_coeff(f, dfs, a, b, n), from_nat[Real](n.suc),
                (b - x0).pow(n))
            (taylor_aux_coeff(f, dfs, a, b, n) * from_nat[Real](n.suc)) * (b - x0).pow(n) =
                taylor_aux_coeff(f, dfs, a, b, n) *
                    (from_nat[Real](n.suc) * (b - x0).pow(n))
            taylor_aux_coeff(f, dfs, a, b, n) *
                (-(from_nat[Real](n.suc)) * (b - x0).pow(n)) =
                -((taylor_aux_coeff(f, dfs, a, b, n) * from_nat[Real](n.suc)) *
                    (b - x0).pow(n))
            neg_neg(taylor_aux_coeff(f, dfs, a, b, n) * from_nat[Real](n.suc) * (b - x0).pow(n))
            -(-(taylor_aux_coeff(f, dfs, a, b, n) * from_nat[Real](n.suc) * (b - x0).pow(n))) =
                taylor_aux_coeff(f, dfs, a, b, n) * from_nat[Real](n.suc) * (b - x0).pow(n)
            -(taylor_aux_coeff(f, dfs, a, b, n) *
                (-(from_nat[Real](n.suc)) * (b - x0).pow(n))) =
                taylor_aux_coeff(f, dfs, a, b, n) * from_nat[Real](n.suc) * (b - x0).pow(n)
            has_derivative_at(pointwise_neg(pointwise_mul(constant[Real, Real](
                        taylor_aux_coeff(f, dfs, a, b, n)),
                    taylor_shift_pow(b, n.suc))), x0,
                taylor_aux_coeff(f, dfs, a, b, n) * from_nat[Real](n.suc) * (b - x0).pow(n))
            derivative_pointwise_add(pointwise_add(constant[Real, Real](f(b)),
                    pointwise_neg(taylor_bt_sum_fn(dfs, b, n.suc))),
                pointwise_neg(pointwise_mul(constant[Real, Real](taylor_aux_coeff(f, dfs, a, b, n)),
                    taylor_shift_pow(b, n.suc))), x0,
                Real.0 + -(taylor_bt_sum_deriv(dfs, b, x0, n.suc)),
                taylor_aux_coeff(f, dfs, a, b, n) * from_nat[Real](n.suc) * (b - x0).pow(n))
            has_derivative_at(pointwise_add(pointwise_add(constant[Real, Real](f(b)),
                        pointwise_neg(taylor_bt_sum_fn(dfs, b, n.suc))),
                    pointwise_neg(pointwise_mul(constant[Real, Real](taylor_aux_coeff(f, dfs, a, b, n)),
                        taylor_shift_pow(b, n.suc)))), x0,
                (Real.0 + -(taylor_bt_sum_deriv(dfs, b, x0, n.suc))) +
                    (taylor_aux_coeff(f, dfs, a, b, n) * from_nat[Real](n.suc) * (b - x0).pow(n)))
            add_zero_left(-(taylor_bt_sum_deriv(dfs, b, x0, n.suc)))
            Real.0 + -(taylor_bt_sum_deriv(dfs, b, x0, n.suc)) =
                -(taylor_bt_sum_deriv(dfs, b, x0, n.suc))
            (Real.0 + -(taylor_bt_sum_deriv(dfs, b, x0, n.suc))) +
                (taylor_aux_coeff(f, dfs, a, b, n) * from_nat[Real](n.suc) * (b - x0).pow(n)) =
                -(taylor_bt_sum_deriv(dfs, b, x0, n.suc)) +
                    (taylor_aux_coeff(f, dfs, a, b, n) * from_nat[Real](n.suc) * (b - x0).pow(n))
            real_mul_comm(taylor_aux_coeff(f, dfs, a, b, n), from_nat[Real](n.suc))
            taylor_aux_coeff(f, dfs, a, b, n) * from_nat[Real](n.suc) =
                from_nat[Real](n.suc) * taylor_aux_coeff(f, dfs, a, b, n)
            (taylor_aux_coeff(f, dfs, a, b, n) * from_nat[Real](n.suc)) * (b - x0).pow(n) =
                (from_nat[Real](n.suc) * taylor_aux_coeff(f, dfs, a, b, n)) * (b - x0).pow(n)
            mul_assoc(from_nat[Real](n.suc), taylor_aux_coeff(f, dfs, a, b, n), (b - x0).pow(n))
            (from_nat[Real](n.suc) * taylor_aux_coeff(f, dfs, a, b, n)) * (b - x0).pow(n) =
                from_nat[Real](n.suc) * (taylor_aux_coeff(f, dfs, a, b, n) * (b - x0).pow(n))
            taylor_aux_coeff(f, dfs, a, b, n) * from_nat[Real](n.suc) * (b - x0).pow(n) =
                from_nat[Real](n.suc) * taylor_aux_coeff(f, dfs, a, b, n) * (b - x0).pow(n)
            (Real.0 + -(taylor_bt_sum_deriv(dfs, b, x0, n.suc))) +
                (taylor_aux_coeff(f, dfs, a, b, n) * from_nat[Real](n.suc) * (b - x0).pow(n)) =
                -(taylor_bt_sum_deriv(dfs, b, x0, n.suc)) +
                    from_nat[Real](n.suc) * taylor_aux_coeff(f, dfs, a, b, n) * (b - x0).pow(n)
            has_derivative_at(pointwise_add(pointwise_add(constant[Real, Real](f(b)),
                        pointwise_neg(taylor_bt_sum_fn(dfs, b, n.suc))),
                    pointwise_neg(pointwise_mul(constant[Real, Real](taylor_aux_coeff(f, dfs, a, b, n)),
                        taylor_shift_pow(b, n.suc)))), x0,
                -(taylor_bt_sum_deriv(dfs, b, x0, n.suc)) +
                    from_nat[Real](n.suc) * taylor_aux_coeff(f, dfs, a, b, n) * (b - x0).pow(n))
            taylor_aux_deriv_val(f, dfs, a, b, n, x0) =
                -partial(taylor_bt_term_deriv(dfs, b, x0), n.suc) +
                    from_nat[Real](n.suc) * taylor_aux_coeff(f, dfs, a, b, n) * (b - x0).pow(n)
            taylor_bt_sum_deriv(dfs, b, x0, n.suc) =
                partial(taylor_bt_term_deriv(dfs, b, x0), n.suc)
            taylor_aux_deriv_val(f, dfs, a, b, n, x0) =
                -(taylor_bt_sum_deriv(dfs, b, x0, n.suc)) +
                    from_nat[Real](n.suc) * taylor_aux_coeff(f, dfs, a, b, n) * (b - x0).pow(n)
            has_derivative_at(pointwise_add(pointwise_add(constant[Real, Real](f(b)),
                        pointwise_neg(taylor_bt_sum_fn(dfs, b, n.suc))),
                    pointwise_neg(pointwise_mul(constant[Real, Real](taylor_aux_coeff(f, dfs, a, b, n)),
                        taylor_shift_pow(b, n.suc)))), x0,
                taylor_aux_deriv_val(f, dfs, a, b, n, x0))
            taylor_aux_fn(f, dfs, a, b, n) = pointwise_add(pointwise_add(constant[Real, Real](f(b)),
                    pointwise_neg(taylor_bt_sum_fn(dfs, b, n.suc))),
                pointwise_neg(pointwise_mul(constant[Real, Real](taylor_aux_coeff(f, dfs, a, b, n)),
                    taylor_shift_pow(b, n.suc))))
            function_eq_transport_predicate_rev(
                function(h: Real -> Real) {
                    has_derivative_at(h, x0, taylor_aux_deriv_val(f, dfs, a, b, n, x0))
                },
                taylor_aux_fn(f, dfs, a, b, n),
                pointwise_add(pointwise_add(constant[Real, Real](f(b)),
                        pointwise_neg(taylor_bt_sum_fn(dfs, b, n.suc))),
                    pointwise_neg(pointwise_mul(constant[Real, Real](taylor_aux_coeff(f, dfs, a, b, n)),
                        taylor_shift_pow(b, n.suc)))))
            has_derivative_at(taylor_aux_fn(f, dfs, a, b, n), x0,
                taylor_aux_deriv_val(f, dfs, a, b, n, x0))
            taylor_aux_deriv_fn(f, dfs, a, b, n, x0) =
                taylor_aux_deriv_val(f, dfs, a, b, n, x0)
            has_derivative_at(taylor_aux_fn(f, dfs, a, b, n), x0,
                taylor_aux_deriv_fn(f, dfs, a, b, n, x0))
        }
        is_derivative_fn_iff(taylor_aux_fn(f, dfs, a, b, n), taylor_aux_deriv_fn(f, dfs, a, b, n))
        is_derivative_fn(taylor_aux_fn(f, dfs, a, b, n), taylor_aux_deriv_fn(f, dfs, a, b, n))
    }
}

/// The partial sum of a negated function is the negation of the partial sum.
theorem taylor_partial_neg(f: Nat -> Real, n: Nat) {
    partial(pointwise_neg(f), n) = -partial(f, n)
} by {
    partial_scalar_mul(-Real.1, f, n)
    -Real.1 * partial(f, n) = partial(mul_fn[Nat, Real](-Real.1, f), n)
    mul_neg_left(Real.1, partial(f, n))
    -Real.1 * partial(f, n) = -(Real.1 * partial(f, n))
    mul_one_left(partial(f, n))
    Real.1 * partial(f, n) = partial(f, n)
    -(Real.1 * partial(f, n)) = -partial(f, n)
    -Real.1 * partial(f, n) = -partial(f, n)
    partial(mul_fn[Nat, Real](-Real.1, f), n) = -partial(f, n)
    forall(k: Nat) {
        if k < n {
            mul_fn[Nat, Real](-Real.1, f, k) = -Real.1 * f(k)
            mul_neg_left(Real.1, f(k))
            -Real.1 * f(k) = -(Real.1 * f(k))
            mul_one_left(f(k))
            Real.1 * f(k) = f(k)
            -(Real.1 * f(k)) = -f(k)
            -Real.1 * f(k) = -f(k)
            mul_fn[Nat, Real](-Real.1, f, k) = -f(k)
            pointwise_neg(f, k) = -f(k)
            mul_fn[Nat, Real](-Real.1, f, k) = pointwise_neg(f, k)
        }
    }
    partial_pointwise_eq(mul_fn[Nat, Real](-Real.1, f), pointwise_neg(f), n)
    partial(mul_fn[Nat, Real](-Real.1, f), n) = partial(pointwise_neg(f), n)
    partial(pointwise_neg(f), n) = -partial(f, n)
}

/// The sum of the first n + 1 derivative terms of the backward Taylor
/// expansion telescopes to the single (n + 1)-st derivative term.
theorem taylor_deriv_sum_telescopes(dfs: Nat -> Real -> Real, b: Real, x: Real, n: Nat) {
    partial(taylor_bt_term_deriv(dfs, b, x), n.suc) =
        (dfs(n.suc, x) / from_nat[Real](n.factorial)) * (b - x).pow(n)
} by {
    define dterm(k: Nat) -> Real {
        (dfs(k.suc, x) / from_nat[Real](k.factorial)) * (b - x).pow(k)
    }
    define u2(k: Nat) -> Real {
        -((dfs(k, x) / from_nat[Real](k.factorial)) * from_nat[Real](k) * (b - x).pow(k - Nat.1))
    }
    forall(k: Nat) {
        if k < n.suc {
            taylor_bt_term_deriv(dfs, b, x, k) =
                (dfs(k.suc, x) / from_nat[Real](k.factorial)) * (b - x).pow(k) -
                    (dfs(k, x) / from_nat[Real](k.factorial)) * from_nat[Real](k) * (b - x).pow(k - Nat.1)
            dterm(k) = (dfs(k.suc, x) / from_nat[Real](k.factorial)) * (b - x).pow(k)
            u2(k) = -((dfs(k, x) / from_nat[Real](k.factorial)) * from_nat[Real](k) * (b - x).pow(k - Nat.1))
            taylor_bt_term_deriv(dfs, b, x, k) = dterm(k) + u2(k)
            add_fn(dterm, u2, k) = dterm(k) + u2(k)
            taylor_bt_term_deriv(dfs, b, x, k) = add_fn(dterm, u2, k)
        }
    }
    partial_pointwise_eq(taylor_bt_term_deriv(dfs, b, x), add_fn(dterm, u2), n.suc)
    partial(taylor_bt_term_deriv(dfs, b, x), n.suc) = partial(add_fn(dterm, u2), n.suc)
    partial_add(dterm, u2, n.suc)
    partial(dterm, n.suc) + partial(u2, n.suc) = partial(add_fn(dterm, u2), n.suc)
    partial(taylor_bt_term_deriv(dfs, b, x), n.suc) = partial(dterm, n.suc) + partial(u2, n.suc)
    partial_shift_suc(u2, n)
    u2(Nat.0) + partial(compose(u2, Nat.suc), n) = partial(u2, n.suc)
    u2(Nat.0) = -((dfs(Nat.0, x) / from_nat[Real](Nat.0.factorial)) *
        from_nat[Real](Nat.0) * (b - x).pow(Nat.0 - Nat.1))
    from_nat_zero[Real]
    from_nat[Real](Nat.0) = Real.0
    sub_lt(Nat.0, Nat.1)
    Nat.0 - Nat.1 = Nat.0
    (b - x).pow(Nat.0 - Nat.1) = (b - x).pow(Nat.0)
    (dfs(Nat.0, x) / from_nat[Real](Nat.0.factorial)) * from_nat[Real](Nat.0) *
        (b - x).pow(Nat.0 - Nat.1) =
        (dfs(Nat.0, x) / from_nat[Real](Nat.0.factorial)) * from_nat[Real](Nat.0) *
            (b - x).pow(Nat.0)
    (dfs(Nat.0, x) / from_nat[Real](Nat.0.factorial)) * from_nat[Real](Nat.0) *
        (b - x).pow(Nat.0) =
        (dfs(Nat.0, x) / from_nat[Real](Nat.0.factorial)) * Real.0 * (b - x).pow(Nat.0)
    mul_zero_right(dfs(Nat.0, x) / from_nat[Real](Nat.0.factorial))
    (dfs(Nat.0, x) / from_nat[Real](Nat.0.factorial)) * Real.0 = Real.0
    (dfs(Nat.0, x) / from_nat[Real](Nat.0.factorial)) * Real.0 * (b - x).pow(Nat.0) =
        Real.0 * (b - x).pow(Nat.0)
    mul_zero_left((b - x).pow(Nat.0))
    Real.0 * (b - x).pow(Nat.0) = Real.0
    (dfs(Nat.0, x) / from_nat[Real](Nat.0.factorial)) * Real.0 * (b - x).pow(Nat.0) = Real.0
    (dfs(Nat.0, x) / from_nat[Real](Nat.0.factorial)) * from_nat[Real](Nat.0) *
        (b - x).pow(Nat.0 - Nat.1) = Real.0
    u2(Nat.0) = -Real.0
    neg_zero
    -Real.0 = Real.0
    u2(Nat.0) = Real.0
    forall(j: Nat) {
        if j < n {
            compose(u2, Nat.suc, j) = u2(Nat.suc(j))
            Nat.suc(j) = j.suc
            compose(u2, Nat.suc, j) = u2(j.suc)
            u2(j.suc) = -((dfs(j.suc, x) / from_nat[Real](j.suc.factorial)) *
                from_nat[Real](j.suc) * (b - x).pow(j.suc - Nat.1))
            suc_sub_one(j)
            j.suc - Nat.1 = j
            (b - x).pow(j.suc - Nat.1) = (b - x).pow(j)
            (dfs(j.suc, x) / from_nat[Real](j.suc.factorial)) * from_nat[Real](j.suc) *
                (b - x).pow(j.suc - Nat.1) =
                (dfs(j.suc, x) / from_nat[Real](j.suc.factorial)) * from_nat[Real](j.suc) *
                    (b - x).pow(j)
            taylor_ratio_mul(j, dfs(j.suc, x))
            (dfs(j.suc, x) / from_nat[Real](j.suc.factorial)) * from_nat[Real](j.suc) =
                dfs(j.suc, x) / from_nat[Real](j.factorial)
            (dfs(j.suc, x) / from_nat[Real](j.suc.factorial)) * from_nat[Real](j.suc) *
                (b - x).pow(j) =
                (dfs(j.suc, x) / from_nat[Real](j.factorial)) * (b - x).pow(j)
            dterm(j) = (dfs(j.suc, x) / from_nat[Real](j.factorial)) * (b - x).pow(j)
            (dfs(j.suc, x) / from_nat[Real](j.suc.factorial)) * from_nat[Real](j.suc) *
                (b - x).pow(j.suc - Nat.1) = dterm(j)
            u2(j.suc) = -dterm(j)
            compose(u2, Nat.suc, j) = -dterm(j)
            pointwise_neg(dterm, j) = -dterm(j)
            compose(u2, Nat.suc, j) = pointwise_neg(dterm, j)
        }
    }
    partial_pointwise_eq(compose(u2, Nat.suc), pointwise_neg(dterm), n)
    partial(compose(u2, Nat.suc), n) = partial(pointwise_neg(dterm), n)
    taylor_partial_neg(dterm, n)
    partial(pointwise_neg(dterm), n) = -partial(dterm, n)
    partial(compose(u2, Nat.suc), n) = -partial(dterm, n)
    u2(Nat.0) + partial(compose(u2, Nat.suc), n) = Real.0 + -partial(dterm, n)
    add_zero_left(-partial(dterm, n))
    Real.0 + -partial(dterm, n) = -partial(dterm, n)
    partial(u2, n.suc) = -partial(dterm, n)
    partial_split_last(dterm, n)
    partial(dterm, n.suc) = partial(dterm, n) + dterm(n)
    partial(taylor_bt_term_deriv(dfs, b, x), n.suc) =
        (partial(dterm, n) + dterm(n)) + -partial(dterm, n)
    add_assoc(partial(dterm, n), dterm(n), -partial(dterm, n))
    (partial(dterm, n) + dterm(n)) + -partial(dterm, n) =
        partial(dterm, n) + (dterm(n) + -partial(dterm, n))
    add_comm(dterm(n), -partial(dterm, n))
    dterm(n) + -partial(dterm, n) = -partial(dterm, n) + dterm(n)
    partial(dterm, n) + (dterm(n) + -partial(dterm, n)) =
        partial(dterm, n) + (-partial(dterm, n) + dterm(n))
    add_assoc(partial(dterm, n), -partial(dterm, n), dterm(n))
    (partial(dterm, n) + -partial(dterm, n)) + dterm(n) =
        partial(dterm, n) + (-partial(dterm, n) + dterm(n))
    add_neg_eq_zero(partial(dterm, n))
    partial(dterm, n) + -partial(dterm, n) = Real.0
    (partial(dterm, n) + -partial(dterm, n)) + dterm(n) = Real.0 + dterm(n)
    add_zero_left(dterm(n))
    Real.0 + dterm(n) = dterm(n)
    (partial(dterm, n) + -partial(dterm, n)) + dterm(n) = dterm(n)
    partial(dterm, n) + (-partial(dterm, n) + dterm(n)) = dterm(n)
    partial(dterm, n) + (dterm(n) + -partial(dterm, n)) = dterm(n)
    (partial(dterm, n) + dterm(n)) + -partial(dterm, n) = dterm(n)
    partial(taylor_bt_term_deriv(dfs, b, x), n.suc) = dterm(n)
    dterm(n) = (dfs(n.suc, x) / from_nat[Real](n.factorial)) * (b - x).pow(n)
    partial(taylor_bt_term_deriv(dfs, b, x), n.suc) =
        (dfs(n.suc, x) / from_nat[Real](n.factorial)) * (b - x).pow(n)
}

/// The raw derivative of the auxiliary function telescopes: all the summands
/// cancel except the (n + 1)-st derivative term.
theorem taylor_aux_deriv_telescopes(f: Real -> Real, dfs: Nat -> Real -> Real, a: Real, b: Real, n: Nat, x: Real) {
    taylor_aux_deriv_val(f, dfs, a, b, n, x) =
        (b - x).pow(n) * (from_nat[Real](n.suc) * taylor_aux_coeff(f, dfs, a, b, n) -
            dfs(n.suc, x) / from_nat[Real](n.factorial))
} by {
    taylor_aux_deriv_val(f, dfs, a, b, n, x) =
        -partial(taylor_bt_term_deriv(dfs, b, x), n.suc) +
            from_nat[Real](n.suc) * taylor_aux_coeff(f, dfs, a, b, n) * (b - x).pow(n)
    taylor_deriv_sum_telescopes(dfs, b, x, n)
    partial(taylor_bt_term_deriv(dfs, b, x), n.suc) =
        (dfs(n.suc, x) / from_nat[Real](n.factorial)) * (b - x).pow(n)
    taylor_aux_deriv_val(f, dfs, a, b, n, x) =
        -((dfs(n.suc, x) / from_nat[Real](n.factorial)) * (b - x).pow(n)) +
            from_nat[Real](n.suc) * taylor_aux_coeff(f, dfs, a, b, n) * (b - x).pow(n)
    mul_neg_left(dfs(n.suc, x) / from_nat[Real](n.factorial), (b - x).pow(n))
    -(dfs(n.suc, x) / from_nat[Real](n.factorial)) * (b - x).pow(n) =
        -((dfs(n.suc, x) / from_nat[Real](n.factorial)) * (b - x).pow(n))
    taylor_aux_deriv_val(f, dfs, a, b, n, x) =
        -(dfs(n.suc, x) / from_nat[Real](n.factorial)) * (b - x).pow(n) +
            from_nat[Real](n.suc) * taylor_aux_coeff(f, dfs, a, b, n) * (b - x).pow(n)
    add_comm(-(dfs(n.suc, x) / from_nat[Real](n.factorial)) * (b - x).pow(n),
        from_nat[Real](n.suc) * taylor_aux_coeff(f, dfs, a, b, n) * (b - x).pow(n))
    -(dfs(n.suc, x) / from_nat[Real](n.factorial)) * (b - x).pow(n) +
        from_nat[Real](n.suc) * taylor_aux_coeff(f, dfs, a, b, n) * (b - x).pow(n) =
        from_nat[Real](n.suc) * taylor_aux_coeff(f, dfs, a, b, n) * (b - x).pow(n) +
            -(dfs(n.suc, x) / from_nat[Real](n.factorial)) * (b - x).pow(n)
    taylor_aux_deriv_val(f, dfs, a, b, n, x) =
        from_nat[Real](n.suc) * taylor_aux_coeff(f, dfs, a, b, n) * (b - x).pow(n) +
            -(dfs(n.suc, x) / from_nat[Real](n.factorial)) * (b - x).pow(n)
    mul_distrib_left(from_nat[Real](n.suc) * taylor_aux_coeff(f, dfs, a, b, n),
        -(dfs(n.suc, x) / from_nat[Real](n.factorial)), (b - x).pow(n))
    (from_nat[Real](n.suc) * taylor_aux_coeff(f, dfs, a, b, n) +
        -(dfs(n.suc, x) / from_nat[Real](n.factorial))) * (b - x).pow(n) =
        from_nat[Real](n.suc) * taylor_aux_coeff(f, dfs, a, b, n) * (b - x).pow(n) +
            -(dfs(n.suc, x) / from_nat[Real](n.factorial)) * (b - x).pow(n)
    from_nat[Real](n.suc) * taylor_aux_coeff(f, dfs, a, b, n) +
        -(dfs(n.suc, x) / from_nat[Real](n.factorial)) =
        from_nat[Real](n.suc) * taylor_aux_coeff(f, dfs, a, b, n) -
            dfs(n.suc, x) / from_nat[Real](n.factorial)
    (from_nat[Real](n.suc) * taylor_aux_coeff(f, dfs, a, b, n) -
        dfs(n.suc, x) / from_nat[Real](n.factorial)) * (b - x).pow(n) =
        from_nat[Real](n.suc) * taylor_aux_coeff(f, dfs, a, b, n) * (b - x).pow(n) +
            -(dfs(n.suc, x) / from_nat[Real](n.factorial)) * (b - x).pow(n)
    taylor_aux_deriv_val(f, dfs, a, b, n, x) =
        (from_nat[Real](n.suc) * taylor_aux_coeff(f, dfs, a, b, n) -
            dfs(n.suc, x) / from_nat[Real](n.factorial)) * (b - x).pow(n)
    real_mul_comm(from_nat[Real](n.suc) * taylor_aux_coeff(f, dfs, a, b, n) -
            dfs(n.suc, x) / from_nat[Real](n.factorial), (b - x).pow(n))
    (from_nat[Real](n.suc) * taylor_aux_coeff(f, dfs, a, b, n) -
        dfs(n.suc, x) / from_nat[Real](n.factorial)) * (b - x).pow(n) =
        (b - x).pow(n) * (from_nat[Real](n.suc) * taylor_aux_coeff(f, dfs, a, b, n) -
            dfs(n.suc, x) / from_nat[Real](n.factorial))
    taylor_aux_deriv_val(f, dfs, a, b, n, x) =
        (b - x).pow(n) * (from_nat[Real](n.suc) * taylor_aux_coeff(f, dfs, a, b, n) -
            dfs(n.suc, x) / from_nat[Real](n.factorial))
}

/// Taylor's theorem of order n with Lagrange remainder: if dfs is a chain of
/// n + 1 derivative functions for f, then the value of f at b equals the n-th
/// order Taylor polynomial of f at a plus the Lagrange remainder
/// f^(n + 1)(c) (b - a)^(n + 1) / (n + 1)! for some interior point c.
theorem taylor_general(n: Nat, f: Real -> Real, dfs: Nat -> Real -> Real, a: Real, b: Real) {
    a < b and is_derivative_chain(f, dfs, n.suc)
    implies exists(c: Real) {
        a < c and c < b and
        f(b) = taylor_poly(f, dfs, a, b, n) + taylor_remainder(f, dfs, a, b, c, n)
    }
} by {
    if a < b and is_derivative_chain(f, dfs, n.suc) {
        taylor_aux_fn_is_derivative_fn(f, dfs, a, b, n)
        is_derivative_fn(taylor_aux_fn(f, dfs, a, b, n), taylor_aux_deriv_fn(f, dfs, a, b, n))
        is_derivative_fn_imp_continuous_on_closed(taylor_aux_fn(f, dfs, a, b, n),
            taylor_aux_deriv_fn(f, dfs, a, b, n), a, b)
        continuous_on_closed(taylor_aux_fn(f, dfs, a, b, n), a, b)
        is_derivative_fn_imp_is_derivative_on_open(taylor_aux_fn(f, dfs, a, b, n),
            taylor_aux_deriv_fn(f, dfs, a, b, n), a, b)
        is_derivative_on_open(taylor_aux_fn(f, dfs, a, b, n),
            taylor_aux_deriv_fn(f, dfs, a, b, n), a, b)
        is_derivative_on_open_imp_differentiable_on_open(taylor_aux_fn(f, dfs, a, b, n),
            taylor_aux_deriv_fn(f, dfs, a, b, n), a, b)
        differentiable_on_open(taylor_aux_fn(f, dfs, a, b, n), a, b)
        taylor_chain_first(f, dfs, n.suc)
        is_derivative_chain(f, dfs, n.suc) implies dfs(Nat.0) = f
        dfs(Nat.0) = f
        taylor_aux_endpoints_equal(f, dfs, a, b, n)
        taylor_aux_fn(f, dfs, a, b, n, a) = taylor_aux_fn(f, dfs, a, b, n, b)
        rolle_theorem(taylor_aux_fn(f, dfs, a, b, n), a, b)
        let c1: Real satisfy {
            a < c1 and c1 < b and
            has_derivative_at(taylor_aux_fn(f, dfs, a, b, n), c1, Real.0)
        }
        a < c1 and c1 < b
        has_derivative_at(taylor_aux_fn(f, dfs, a, b, n), c1, Real.0)
        is_derivative_fn_at(taylor_aux_fn(f, dfs, a, b, n),
            taylor_aux_deriv_fn(f, dfs, a, b, n), c1)
        has_derivative_at(taylor_aux_fn(f, dfs, a, b, n), c1,
            taylor_aux_deriv_fn(f, dfs, a, b, n, c1))
        has_derivative_at_unique(taylor_aux_fn(f, dfs, a, b, n), c1, Real.0,
            taylor_aux_deriv_fn(f, dfs, a, b, n, c1))
        Real.0 = taylor_aux_deriv_fn(f, dfs, a, b, n, c1)
        taylor_aux_deriv_fn(f, dfs, a, b, n, c1) = Real.0
        taylor_aux_deriv_fn(f, dfs, a, b, n, c1) =
            taylor_aux_deriv_val(f, dfs, a, b, n, c1)
        taylor_aux_deriv_val(f, dfs, a, b, n, c1) = Real.0
        taylor_aux_deriv_telescopes(f, dfs, a, b, n, c1)
        taylor_aux_deriv_val(f, dfs, a, b, n, c1) =
            (b - c1).pow(n) * (from_nat[Real](n.suc) * taylor_aux_coeff(f, dfs, a, b, n) -
                dfs(n.suc, c1) / from_nat[Real](n.factorial))
        (b - c1).pow(n) * (from_nat[Real](n.suc) * taylor_aux_coeff(f, dfs, a, b, n) -
            dfs(n.suc, c1) / from_nat[Real](n.factorial)) = Real.0
        lt_imp_ne_symm(c1, b)
        b != c1
        sub_ne_zero_of_ne(b, c1)
        b - c1 != Real.0
        pow_not_zero[Real](b - c1, n)
        (b - c1).pow(n) != Real.0
        mul_zero_right(from_nat[Real](n.suc) * taylor_aux_coeff(f, dfs, a, b, n) -
            dfs(n.suc, c1) / from_nat[Real](n.factorial))
        (from_nat[Real](n.suc) * taylor_aux_coeff(f, dfs, a, b, n) -
            dfs(n.suc, c1) / from_nat[Real](n.factorial)) * Real.0 = Real.0
        mul_left_cancel((b - c1).pow(n),
            from_nat[Real](n.suc) * taylor_aux_coeff(f, dfs, a, b, n) -
                dfs(n.suc, c1) / from_nat[Real](n.factorial), Real.0)
        from_nat[Real](n.suc) * taylor_aux_coeff(f, dfs, a, b, n) -
            dfs(n.suc, c1) / from_nat[Real](n.factorial) = Real.0
        sub_zero_imp_eq(from_nat[Real](n.suc) * taylor_aux_coeff(f, dfs, a, b, n),
            dfs(n.suc, c1) / from_nat[Real](n.factorial))
        from_nat[Real](n.suc) * taylor_aux_coeff(f, dfs, a, b, n) =
            dfs(n.suc, c1) / from_nat[Real](n.factorial)
        from_nat_suc_ne_zero(n)
        from_nat[Real](n.suc) != Real.0
        (from_nat[Real](n.suc) * taylor_aux_coeff(f, dfs, a, b, n)) * from_nat[Real](n.suc).inverse =
            (dfs(n.suc, c1) / from_nat[Real](n.factorial)) * from_nat[Real](n.suc).inverse
        mul_inverse(from_nat[Real](n.suc))
        from_nat[Real](n.suc) * from_nat[Real](n.suc).inverse = Real.1
        mul_assoc(from_nat[Real](n.suc), taylor_aux_coeff(f, dfs, a, b, n),
            from_nat[Real](n.suc).inverse)
        (from_nat[Real](n.suc) * taylor_aux_coeff(f, dfs, a, b, n)) * from_nat[Real](n.suc).inverse =
            from_nat[Real](n.suc) * (taylor_aux_coeff(f, dfs, a, b, n) * from_nat[Real](n.suc).inverse)
        real_mul_comm(taylor_aux_coeff(f, dfs, a, b, n), from_nat[Real](n.suc).inverse)
        taylor_aux_coeff(f, dfs, a, b, n) * from_nat[Real](n.suc).inverse =
            from_nat[Real](n.suc).inverse * taylor_aux_coeff(f, dfs, a, b, n)
        from_nat[Real](n.suc) * (taylor_aux_coeff(f, dfs, a, b, n) * from_nat[Real](n.suc).inverse) =
            from_nat[Real](n.suc) * (from_nat[Real](n.suc).inverse * taylor_aux_coeff(f, dfs, a, b, n))
        mul_assoc(from_nat[Real](n.suc), from_nat[Real](n.suc).inverse,
            taylor_aux_coeff(f, dfs, a, b, n))
        from_nat[Real](n.suc) * (from_nat[Real](n.suc).inverse * taylor_aux_coeff(f, dfs, a, b, n)) =
            (from_nat[Real](n.suc) * from_nat[Real](n.suc).inverse) * taylor_aux_coeff(f, dfs, a, b, n)
        (from_nat[Real](n.suc) * from_nat[Real](n.suc).inverse) * taylor_aux_coeff(f, dfs, a, b, n) =
            Real.1 * taylor_aux_coeff(f, dfs, a, b, n)
        mul_one_left(taylor_aux_coeff(f, dfs, a, b, n))
        Real.1 * taylor_aux_coeff(f, dfs, a, b, n) = taylor_aux_coeff(f, dfs, a, b, n)
        from_nat[Real](n.suc) * (taylor_aux_coeff(f, dfs, a, b, n) * from_nat[Real](n.suc).inverse) =
            taylor_aux_coeff(f, dfs, a, b, n)
        (from_nat[Real](n.suc) * taylor_aux_coeff(f, dfs, a, b, n)) * from_nat[Real](n.suc).inverse =
            taylor_aux_coeff(f, dfs, a, b, n)
        taylor_aux_coeff(f, dfs, a, b, n) =
            (dfs(n.suc, c1) / from_nat[Real](n.factorial)) * from_nat[Real](n.suc).inverse
        (dfs(n.suc, c1) / from_nat[Real](n.factorial)) / from_nat[Real](n.suc) =
            (dfs(n.suc, c1) / from_nat[Real](n.factorial)) * from_nat[Real](n.suc).inverse
        taylor_aux_coeff(f, dfs, a, b, n) =
            (dfs(n.suc, c1) / from_nat[Real](n.factorial)) / from_nat[Real](n.suc)
        taylor_ratio_div(n, dfs(n.suc, c1))
        (dfs(n.suc, c1) / from_nat[Real](n.factorial)) / from_nat[Real](n.suc) =
            dfs(n.suc, c1) / from_nat[Real](n.suc.factorial)
        taylor_aux_coeff(f, dfs, a, b, n) =
            dfs(n.suc, c1) / from_nat[Real](n.suc.factorial)
        taylor_aux_at_a_zero(f, dfs, a, b, n)
        taylor_aux_val(f, dfs, a, b, n, a) = Real.0
        taylor_aux_val(f, dfs, a, b, n, a) = f(b) - partial(taylor_bt_term(dfs, b, a), n.suc) -
            taylor_aux_coeff(f, dfs, a, b, n) * (b - a).pow(n.suc)
        f(b) - partial(taylor_bt_term(dfs, b, a), n.suc) -
            taylor_aux_coeff(f, dfs, a, b, n) * (b - a).pow(n.suc) = Real.0
        sub_zero_imp_eq(f(b) - partial(taylor_bt_term(dfs, b, a), n.suc),
            taylor_aux_coeff(f, dfs, a, b, n) * (b - a).pow(n.suc))
        f(b) - partial(taylor_bt_term(dfs, b, a), n.suc) =
            taylor_aux_coeff(f, dfs, a, b, n) * (b - a).pow(n.suc)
        taylor2_sub_add_cancel(f(b), partial(taylor_bt_term(dfs, b, a), n.suc))
        (f(b) - partial(taylor_bt_term(dfs, b, a), n.suc)) +
            partial(taylor_bt_term(dfs, b, a), n.suc) = f(b)
        (f(b) - partial(taylor_bt_term(dfs, b, a), n.suc)) +
            partial(taylor_bt_term(dfs, b, a), n.suc) =
            taylor_aux_coeff(f, dfs, a, b, n) * (b - a).pow(n.suc) +
                partial(taylor_bt_term(dfs, b, a), n.suc)
        f(b) = taylor_aux_coeff(f, dfs, a, b, n) * (b - a).pow(n.suc) +
            partial(taylor_bt_term(dfs, b, a), n.suc)
        add_comm(taylor_aux_coeff(f, dfs, a, b, n) * (b - a).pow(n.suc),
            partial(taylor_bt_term(dfs, b, a), n.suc))
        f(b) = partial(taylor_bt_term(dfs, b, a), n.suc) +
            taylor_aux_coeff(f, dfs, a, b, n) * (b - a).pow(n.suc)
        taylor_aux_coeff(f, dfs, a, b, n) * (b - a).pow(n.suc) =
            (dfs(n.suc, c1) / from_nat[Real](n.suc.factorial)) * (b - a).pow(n.suc)
        f(b) = partial(taylor_bt_term(dfs, b, a), n.suc) +
            (dfs(n.suc, c1) / from_nat[Real](n.suc.factorial)) * (b - a).pow(n.suc)
        taylor_poly(f, dfs, a, b, n) = partial(taylor_term(f, dfs, a, b), n.suc)
        forall(k: Nat) {
            if k < n.suc {
                taylor_bt_term(dfs, b, a, k) =
                    (dfs(k, a) / from_nat[Real](k.factorial)) * (b - a).pow(k)
                taylor_term(f, dfs, a, b, k) =
                    (iterated_derivative(f, dfs, k, a) / from_nat[Real](k.factorial)) *
                        (b - a).pow(k)
                iterated_derivative(f, dfs, k, a) = dfs(k, a)
                taylor_term(f, dfs, a, b, k) =
                    (dfs(k, a) / from_nat[Real](k.factorial)) * (b - a).pow(k)
                taylor_bt_term(dfs, b, a, k) = taylor_term(f, dfs, a, b, k)
            }
        }
        partial_pointwise_eq(taylor_bt_term(dfs, b, a), taylor_term(f, dfs, a, b), n.suc)
        partial(taylor_bt_term(dfs, b, a), n.suc) = partial(taylor_term(f, dfs, a, b), n.suc)
        partial(taylor_bt_term(dfs, b, a), n.suc) = taylor_poly(f, dfs, a, b, n)
        taylor_remainder(f, dfs, a, b, c1, n) =
            (iterated_derivative(f, dfs, n.suc, c1) / from_nat[Real](n.suc.factorial)) *
                (b - a).pow(n.suc)
        iterated_derivative(f, dfs, n.suc, c1) = dfs(n.suc, c1)
        taylor_remainder(f, dfs, a, b, c1, n) =
            (dfs(n.suc, c1) / from_nat[Real](n.suc.factorial)) * (b - a).pow(n.suc)
        f(b) = taylor_poly(f, dfs, a, b, n) + taylor_remainder(f, dfs, a, b, c1, n)
        exists(c2: Real) {
            a < c2 and c2 < b and
            f(b) = taylor_poly(f, dfs, a, b, n) + taylor_remainder(f, dfs, a, b, c2, n)
        }
    }
}

/// A natural below one is zero.
theorem taylor_lt_one_cases(k: Nat) {
    k < Nat.1 implies k = Nat.0
}
/// The chain of length one is a derivative chain of length one for f.
theorem taylor_chain1_is_derivative_chain(f: Real -> Real, df: Real -> Real) {
    is_derivative_fn(f, df)
    implies is_derivative_chain(f, taylor_chain1(f, df), Nat.1)
} by {
    if is_derivative_fn(f, df) {
        taylor_chain1_zero(f, df)
        taylor_chain1(f, df, Nat.0) = f
        forall(k: Nat) {
            if k < Nat.1 {
                taylor_lt_one_cases(k)
                k < Nat.1 implies k = Nat.0
                k < Nat.1
                k = Nat.0
                taylor_chain1_zero(f, df)
                taylor_chain1(f, df, Nat.0) = f
                taylor_chain1_suc(f, df, Nat.0)
                taylor_chain1(f, df, Nat.0.suc) = df
                Nat.0.suc = Nat.1
                taylor_chain1(f, df, Nat.1) = df
                is_derivative_fn(taylor_chain1(f, df, Nat.0), taylor_chain1(f, df, Nat.1))
                is_derivative_fn(taylor_chain1(f, df, k), taylor_chain1(f, df, k.suc))
            }
        }
        is_derivative_chain(f, taylor_chain1(f, df), Nat.1) =
            (taylor_chain1(f, df, Nat.0) = f and forall(k: Nat) {
                k < Nat.1 implies is_derivative_fn(taylor_chain1(f, df, k), taylor_chain1(f, df, k.suc))
            })
        is_derivative_chain(f, taylor_chain1(f, df), Nat.1)
    }
}

/// The chain of length two is a derivative chain of length two for f.
theorem taylor_chain2_is_derivative_chain(f: Real -> Real, df: Real -> Real, ddf: Real -> Real) {
    is_derivative_fn(f, df) and is_derivative_fn(df, ddf)
    implies is_derivative_chain(f, taylor_chain2(f, df, ddf), Nat.2)
} by {
    if is_derivative_fn(f, df) and is_derivative_fn(df, ddf) {
        taylor_chain2_zero(f, df, ddf)
        taylor_chain2(f, df, ddf, Nat.0) = f
        forall(k: Nat) {
            if k < Nat.2 {
                lt_suc_right(k, Nat.1)
                if k = Nat.1 {
                    taylor_chain2_one(f, df, ddf)
                    taylor_chain2(f, df, ddf, Nat.1) = df
                    taylor_chain2_suc_suc(f, df, ddf, Nat.0)
                    taylor_chain2(f, df, ddf, Nat.0.suc.suc) = ddf
                    Nat.0.suc.suc = Nat.2
                    taylor_chain2(f, df, ddf, Nat.2) = ddf
                    is_derivative_fn(taylor_chain2(f, df, ddf, Nat.1), taylor_chain2(f, df, ddf, Nat.2))
                    is_derivative_fn(taylor_chain2(f, df, ddf, k), taylor_chain2(f, df, ddf, k.suc))
                } else {
                    k < Nat.1
                    taylor_lt_one_cases(k)
                    k < Nat.1 implies k = Nat.0
                    k = Nat.0
                    taylor_chain2_zero(f, df, ddf)
                    taylor_chain2(f, df, ddf, Nat.0) = f
                    taylor_chain2_one(f, df, ddf)
                    taylor_chain2(f, df, ddf, Nat.1) = df
                    is_derivative_fn(taylor_chain2(f, df, ddf, Nat.0), taylor_chain2(f, df, ddf, Nat.1))
                    is_derivative_fn(taylor_chain2(f, df, ddf, k), taylor_chain2(f, df, ddf, k.suc))
                }
            }
        }
        is_derivative_chain(f, taylor_chain2(f, df, ddf), Nat.2) =
            (taylor_chain2(f, df, ddf, Nat.0) = f and forall(k: Nat) {
                k < Nat.2 implies is_derivative_fn(taylor_chain2(f, df, ddf, k),
                    taylor_chain2(f, df, ddf, k.suc))
            })
        is_derivative_chain(f, taylor_chain2(f, df, ddf), Nat.2)
    }
}

/// The chain of length three is a derivative chain of length three for f.
theorem taylor_chain3_is_derivative_chain(f: Real -> Real, df: Real -> Real, ddf: Real -> Real, dddf: Real -> Real) {
    is_derivative_fn(f, df) and is_derivative_fn(df, ddf) and is_derivative_fn(ddf, dddf)
    implies is_derivative_chain(f, taylor_chain3(f, df, ddf, dddf), Nat.3)
} by {
    if is_derivative_fn(f, df) and is_derivative_fn(df, ddf) and is_derivative_fn(ddf, dddf) {
        taylor_chain3_zero(f, df, ddf, dddf)
        taylor_chain3(f, df, ddf, dddf, Nat.0) = f
        forall(k: Nat) {
            if k < Nat.3 {
                lt_suc_right(k, Nat.2)
                if k = Nat.2 {
                    taylor_chain3_two(f, df, ddf, dddf)
                    taylor_chain3(f, df, ddf, dddf, Nat.2) = ddf
                    taylor_chain3_three(f, df, ddf, dddf)
                    taylor_chain3(f, df, ddf, dddf, Nat.3) = dddf
                    is_derivative_fn(taylor_chain3(f, df, ddf, dddf, Nat.2),
                        taylor_chain3(f, df, ddf, dddf, Nat.3))
                    is_derivative_fn(taylor_chain3(f, df, ddf, dddf, k),
                        taylor_chain3(f, df, ddf, dddf, k.suc))
                } else {
                    k < Nat.2
                    lt_suc_right(k, Nat.1)
                    if k = Nat.1 {
                        taylor_chain3_one(f, df, ddf, dddf)
                        taylor_chain3(f, df, ddf, dddf, Nat.1) = df
                        taylor_chain3_two(f, df, ddf, dddf)
                        taylor_chain3(f, df, ddf, dddf, Nat.2) = ddf
                        is_derivative_fn(taylor_chain3(f, df, ddf, dddf, Nat.1),
                            taylor_chain3(f, df, ddf, dddf, Nat.2))
                        is_derivative_fn(taylor_chain3(f, df, ddf, dddf, k),
                            taylor_chain3(f, df, ddf, dddf, k.suc))
                    } else {
                        k < Nat.1
                        taylor_lt_one_cases(k)
                        k < Nat.1 implies k = Nat.0
                        k = Nat.0
                        taylor_chain3_zero(f, df, ddf, dddf)
                        taylor_chain3(f, df, ddf, dddf, Nat.0) = f
                        taylor_chain3_one(f, df, ddf, dddf)
                        taylor_chain3(f, df, ddf, dddf, Nat.1) = df
                        is_derivative_fn(taylor_chain3(f, df, ddf, dddf, Nat.0),
                            taylor_chain3(f, df, ddf, dddf, Nat.1))
                        is_derivative_fn(taylor_chain3(f, df, ddf, dddf, k),
                            taylor_chain3(f, df, ddf, dddf, k.suc))
                    }
                }
            }
        }
        is_derivative_chain(f, taylor_chain3(f, df, ddf, dddf), Nat.3) =
            (taylor_chain3(f, df, ddf, dddf, Nat.0) = f and forall(k: Nat) {
                k < Nat.3 implies is_derivative_fn(taylor_chain3(f, df, ddf, dddf, k),
                    taylor_chain3(f, df, ddf, dddf, k.suc))
            })
        is_derivative_chain(f, taylor_chain3(f, df, ddf, dddf), Nat.3)
    }
}

/// The chain of length four is a derivative chain of length four for f.
theorem taylor_chain4_is_derivative_chain(f: Real -> Real, df: Real -> Real, ddf: Real -> Real, dddf: Real -> Real, dddd: Real -> Real) {
    is_derivative_fn(f, df) and is_derivative_fn(df, ddf) and is_derivative_fn(ddf, dddf) and is_derivative_fn(dddf, dddd)
    implies is_derivative_chain(f, taylor_chain4(f, df, ddf, dddf, dddd), Nat.4)
} by {
    if is_derivative_fn(f, df) and is_derivative_fn(df, ddf) and is_derivative_fn(ddf, dddf) and is_derivative_fn(dddf, dddd) {
        taylor_chain4_zero(f, df, ddf, dddf, dddd)
        taylor_chain4(f, df, ddf, dddf, dddd, Nat.0) = f
        forall(k: Nat) {
            if k < Nat.4 {
                lt_suc_right(k, Nat.3)
                if k = Nat.3 {
                    taylor_chain4_three(f, df, ddf, dddf, dddd)
                    taylor_chain4(f, df, ddf, dddf, dddd, Nat.3) = dddf
                    taylor_chain4_four(f, df, ddf, dddf, dddd)
                    taylor_chain4(f, df, ddf, dddf, dddd, Nat.4) = dddd
                    is_derivative_fn(taylor_chain4(f, df, ddf, dddf, dddd, Nat.3),
                        taylor_chain4(f, df, ddf, dddf, dddd, Nat.4))
                    is_derivative_fn(taylor_chain4(f, df, ddf, dddf, dddd, k),
                        taylor_chain4(f, df, ddf, dddf, dddd, k.suc))
                } else {
                    k < Nat.3
                    lt_suc_right(k, Nat.2)
                    if k = Nat.2 {
                        taylor_chain4_two(f, df, ddf, dddf, dddd)
                        taylor_chain4(f, df, ddf, dddf, dddd, Nat.2) = ddf
                        taylor_chain4_three(f, df, ddf, dddf, dddd)
                        taylor_chain4(f, df, ddf, dddf, dddd, Nat.3) = dddf
                        is_derivative_fn(taylor_chain4(f, df, ddf, dddf, dddd, Nat.2),
                            taylor_chain4(f, df, ddf, dddf, dddd, Nat.3))
                        is_derivative_fn(taylor_chain4(f, df, ddf, dddf, dddd, k),
                            taylor_chain4(f, df, ddf, dddf, dddd, k.suc))
                    } else {
                        k < Nat.2
                        lt_suc_right(k, Nat.1)
                        if k = Nat.1 {
                            taylor_chain4_one(f, df, ddf, dddf, dddd)
                            taylor_chain4(f, df, ddf, dddf, dddd, Nat.1) = df
                            taylor_chain4_two(f, df, ddf, dddf, dddd)
                            taylor_chain4(f, df, ddf, dddf, dddd, Nat.2) = ddf
                            is_derivative_fn(taylor_chain4(f, df, ddf, dddf, dddd, Nat.1),
                                taylor_chain4(f, df, ddf, dddf, dddd, Nat.2))
                            is_derivative_fn(taylor_chain4(f, df, ddf, dddf, dddd, k),
                                taylor_chain4(f, df, ddf, dddf, dddd, k.suc))
                        } else {
                            k < Nat.1
                            taylor_lt_one_cases(k)
                            k < Nat.1 implies k = Nat.0
                            k = Nat.0
                            taylor_chain4_zero(f, df, ddf, dddf, dddd)
                            taylor_chain4(f, df, ddf, dddf, dddd, Nat.0) = f
                            taylor_chain4_one(f, df, ddf, dddf, dddd)
                            taylor_chain4(f, df, ddf, dddf, dddd, Nat.1) = df
                            is_derivative_fn(taylor_chain4(f, df, ddf, dddf, dddd, Nat.0),
                                taylor_chain4(f, df, ddf, dddf, dddd, Nat.1))
                            is_derivative_fn(taylor_chain4(f, df, ddf, dddf, dddd, k),
                                taylor_chain4(f, df, ddf, dddf, dddd, k.suc))
                        }
                    }
                }
            }
        }
        is_derivative_chain(f, taylor_chain4(f, df, ddf, dddf, dddd), Nat.4) =
            (taylor_chain4(f, df, ddf, dddf, dddd, Nat.0) = f and forall(k: Nat) {
                k < Nat.4 implies is_derivative_fn(taylor_chain4(f, df, ddf, dddf, dddd, k),
                    taylor_chain4(f, df, ddf, dddf, dddd, k.suc))
            })
        is_derivative_chain(f, taylor_chain4(f, df, ddf, dddf, dddd), Nat.4)
    }
}

/// Taylor's theorem of order zero, derived from the general theorem: this is
/// the statement of taylor_general_zero of taylor_general.ac.
theorem taylor_general_inst_zero(f: Real -> Real, df: Real -> Real, a: Real, b: Real) {
    a < b and is_derivative_fn(f, df)
    implies exists(c: Real) {
        a < c and c < b and
        f(b) = taylor_poly(f, taylor_chain1(f, df), a, b, Nat.0) +
            taylor_remainder(f, taylor_chain1(f, df), a, b, c, Nat.0)
    }
} by {
    if a < b and is_derivative_fn(f, df) {
        taylor_chain1_is_derivative_chain(f, df)
        is_derivative_chain(f, taylor_chain1(f, df), Nat.1)
        taylor_general(Nat.0, f, taylor_chain1(f, df), a, b)
        exists(c: Real) {
            a < c and c < b and
            f(b) = taylor_poly(f, taylor_chain1(f, df), a, b, Nat.0) +
                taylor_remainder(f, taylor_chain1(f, df), a, b, c, Nat.0)
        }
    }
}

/// Taylor's theorem of order one, derived from the general theorem: this is
/// the statement of taylor_general_one of taylor_general.ac.
theorem taylor_general_inst_one(f: Real -> Real, df: Real -> Real, ddf: Real -> Real, a: Real, b: Real) {
    a < b and is_derivative_fn(f, df) and is_derivative_fn(df, ddf)
    implies exists(c: Real) {
        a < c and c < b and
        f(b) = taylor_poly(f, taylor_chain2(f, df, ddf), a, b, Nat.1) +
            taylor_remainder(f, taylor_chain2(f, df, ddf), a, b, c, Nat.1)
    }
} by {
    if a < b and is_derivative_fn(f, df) and is_derivative_fn(df, ddf) {
        taylor_chain2_is_derivative_chain(f, df, ddf)
        is_derivative_chain(f, taylor_chain2(f, df, ddf), Nat.2)
        taylor_general(Nat.1, f, taylor_chain2(f, df, ddf), a, b)
        exists(c: Real) {
            a < c and c < b and
            f(b) = taylor_poly(f, taylor_chain2(f, df, ddf), a, b, Nat.1) +
                taylor_remainder(f, taylor_chain2(f, df, ddf), a, b, c, Nat.1)
        }
    }
}

/// Taylor's theorem of order two, derived from the general theorem: this is
/// the statement of taylor_general_two of taylor_general.ac.
theorem taylor_general_inst_two(f: Real -> Real, df: Real -> Real, ddf: Real -> Real, dddf: Real -> Real, a: Real, b: Real) {
    a < b and is_derivative_fn(f, df) and is_derivative_fn(df, ddf) and is_derivative_fn(ddf, dddf)
    implies exists(c: Real) {
        a < c and c < b and
        f(b) = taylor_poly(f, taylor_chain3(f, df, ddf, dddf), a, b, Nat.2) +
            taylor_remainder(f, taylor_chain3(f, df, ddf, dddf), a, b, c, Nat.2)
    }
} by {
    if a < b and is_derivative_fn(f, df) and is_derivative_fn(df, ddf) and is_derivative_fn(ddf, dddf) {
        taylor_chain3_is_derivative_chain(f, df, ddf, dddf)
        is_derivative_chain(f, taylor_chain3(f, df, ddf, dddf), Nat.3)
        taylor_general(Nat.2, f, taylor_chain3(f, df, ddf, dddf), a, b)
        exists(c: Real) {
            a < c and c < b and
            f(b) = taylor_poly(f, taylor_chain3(f, df, ddf, dddf), a, b, Nat.2) +
                taylor_remainder(f, taylor_chain3(f, df, ddf, dddf), a, b, c, Nat.2)
        }
    }
}

/// Taylor's theorem of order three, derived from the general theorem: this is
/// the statement of taylor_general_three of taylor_general.ac.
theorem taylor_general_inst_three(f: Real -> Real, df: Real -> Real, ddf: Real -> Real, dddf: Real -> Real, dddd: Real -> Real, a: Real, b: Real) {
    a < b and is_derivative_fn(f, df) and is_derivative_fn(df, ddf) and is_derivative_fn(ddf, dddf) and is_derivative_fn(dddf, dddd)
    implies exists(c: Real) {
        a < c and c < b and
        f(b) = taylor_poly(f, taylor_chain4(f, df, ddf, dddf, dddd), a, b, Nat.3) +
            taylor_remainder(f, taylor_chain4(f, df, ddf, dddf, dddd), a, b, c, Nat.3)
    }
} by {
    if a < b and is_derivative_fn(f, df) and is_derivative_fn(df, ddf) and is_derivative_fn(ddf, dddf) and is_derivative_fn(dddf, dddd) {
        taylor_chain4_is_derivative_chain(f, df, ddf, dddf, dddd)
        is_derivative_chain(f, taylor_chain4(f, df, ddf, dddf, dddd), Nat.4)
        taylor_general(Nat.3, f, taylor_chain4(f, df, ddf, dddf, dddd), a, b)
        exists(c: Real) {
            a < c and c < b and
            f(b) = taylor_poly(f, taylor_chain4(f, df, ddf, dddf, dddd), a, b, Nat.3) +
                taylor_remainder(f, taylor_chain4(f, df, ddf, dddf, dddd), a, b, c, Nat.3)
        }
    }
}

/// The exponential chain: every entry of the chain is the exponential function.
define exp_chain(k: Nat) -> Real -> Real {
    Real.exp
}

/// The exponential chain is a derivative chain of length n + 1 for Real.exp, for
/// every n.
theorem exp_chain_is_derivative_chain(n: Nat) {
    is_derivative_chain(Real.exp, exp_chain, n.suc)
} by {
    exp_chain(Nat.0) = Real.exp
    forall(k: Nat) {
        if k < n.suc {
            exp_chain(k) = Real.exp
            exp_chain(k.suc) = Real.exp
            exp_is_derivative_fn
            is_derivative_fn(Real.exp, Real.exp)
            is_derivative_fn(exp_chain(k), exp_chain(k.suc))
        }
    }
    is_derivative_chain(Real.exp, exp_chain, n.suc) = (exp_chain(Nat.0) = Real.exp and forall(k: Nat) {
        k < n.suc implies is_derivative_fn(exp_chain(k), exp_chain(k.suc))
    })
    is_derivative_chain(Real.exp, exp_chain, n.suc)
}

/// The exponential term x^k / k! written with the natural-number embedding.
theorem exp_term_from_nat(x: Real, k: Nat) {
    exp_term(x, k) = x.pow(k) / from_nat[Real](k.factorial)
} by {
    exp_term(x, k) = x.pow(k) / Real.from_rat(Rat.from_nat(k.factorial))
    from_nat_is_from_rat(k.factorial)
    from_nat[Real](k.factorial) = Real.from_rat(Rat.from_nat(k.factorial))
    x.pow(k) / Real.from_rat(Rat.from_nat(k.factorial)) =
        x.pow(k) / from_nat[Real](k.factorial)
    exp_term(x, k) = x.pow(k) / from_nat[Real](k.factorial)
}

/// The exponential function is the limit of its partial sums.
theorem exp_series_eq(x: Real) {
    x.exp = limit(partial(exp_term(x)))
}

/// The Taylor series of Real.exp at zero converges to Real.exp.
theorem taylor_exp_series_converges(x: Real) {
    converges_to(partial(exp_term(x)), x.exp)
} by {
    exp_term_partial_converges(x)
    converges(partial(exp_term(x)))
    converges_imp_converges_to(partial(exp_term(x)))
    converges_to(partial(exp_term(x)), limit(partial(exp_term(x))))
    exp_series_eq(x)
    x.exp = limit(partial(exp_term(x)))
    limit(partial(exp_term(x))) = x.exp
    converges_to(partial(exp_term(x)), x.exp)
}

/// Taylor's theorem for Real.exp at zero with Lagrange remainder: x.exp is its
/// n-th order Taylor polynomial at zero plus the remainder c.exp x^(n + 1)
/// divided by (n + 1)!, for some interior point c.
theorem taylor_exp_remainder(n: Nat, x: Real) {
    Real.0 < x
    implies exists(c: Real) {
        Real.0 < c and c < x and
        x.exp = partial(exp_term(x), n.suc) +
            (c.exp / from_nat[Real](n.suc.factorial)) * x.pow(n.suc)
    }
} by {
    if Real.0 < x {
        exp_chain_is_derivative_chain(n)
        is_derivative_chain(Real.exp, exp_chain, n.suc)
        taylor_general(n, Real.exp, exp_chain, Real.0, x)
        let c: Real satisfy {
            Real.0 < c and c < x and
            x.exp = taylor_poly(Real.exp, exp_chain, Real.0, x, n) +
                taylor_remainder(Real.exp, exp_chain, Real.0, x, c, n)
        }
        Real.0 < c and c < x
        x.exp = taylor_poly(Real.exp, exp_chain, Real.0, x, n) +
            taylor_remainder(Real.exp, exp_chain, Real.0, x, c, n)
        taylor_poly(Real.exp, exp_chain, Real.0, x, n) =
            partial(taylor_term(Real.exp, exp_chain, Real.0, x), n.suc)
        forall(k: Nat) {
            if k < n.suc {
                taylor_term(Real.exp, exp_chain, Real.0, x, k) =
                    (iterated_derivative(Real.exp, exp_chain, k, Real.0) / from_nat[Real](k.factorial)) *
                        (x - Real.0).pow(k)
                iterated_derivative(Real.exp, exp_chain, k, Real.0) = exp_chain(k, Real.0)
                exp_chain(k, Real.0) = (Real.0).exp
                exp_zero
                (Real.0).exp = Real.1
                iterated_derivative(Real.exp, exp_chain, k, Real.0) = Real.1
                x - Real.0 = x
                (x - Real.0).pow(k) = x.pow(k)
                taylor_term(Real.exp, exp_chain, Real.0, x, k) =
                    (Real.1 / from_nat[Real](k.factorial)) * x.pow(k)
                exp_term_from_nat(x, k)
                exp_term(x, k) = x.pow(k) / from_nat[Real](k.factorial)
                x.pow(k) / from_nat[Real](k.factorial) =
                    x.pow(k) * from_nat[Real](k.factorial).inverse
                real_mul_comm(Real.1, from_nat[Real](k.factorial).inverse)
                Real.1 * from_nat[Real](k.factorial).inverse =
                    from_nat[Real](k.factorial).inverse
                (Real.1 / from_nat[Real](k.factorial)) * x.pow(k) =
                    (Real.1 * from_nat[Real](k.factorial).inverse) * x.pow(k)
                (Real.1 * from_nat[Real](k.factorial).inverse) * x.pow(k) =
                    from_nat[Real](k.factorial).inverse * x.pow(k)
                x.pow(k) * from_nat[Real](k.factorial).inverse =
                    from_nat[Real](k.factorial).inverse * x.pow(k)
                (Real.1 / from_nat[Real](k.factorial)) * x.pow(k) =
                    x.pow(k) * from_nat[Real](k.factorial).inverse
                (Real.1 / from_nat[Real](k.factorial)) * x.pow(k) =
                    x.pow(k) / from_nat[Real](k.factorial)
                taylor_term(Real.exp, exp_chain, Real.0, x, k) = exp_term(x, k)
            }
        }
        partial_pointwise_eq(taylor_term(Real.exp, exp_chain, Real.0, x), exp_term(x), n.suc)
        partial(taylor_term(Real.exp, exp_chain, Real.0, x), n.suc) = partial(exp_term(x), n.suc)
        taylor_poly(Real.exp, exp_chain, Real.0, x, n) = partial(exp_term(x), n.suc)
        taylor_remainder(Real.exp, exp_chain, Real.0, x, c, n) =
            (iterated_derivative(Real.exp, exp_chain, n.suc, c) / from_nat[Real](n.suc.factorial)) *
                (x - Real.0).pow(n.suc)
        iterated_derivative(Real.exp, exp_chain, n.suc, c) = exp_chain(n.suc, c)
        exp_chain(n.suc, c) = c.exp
        iterated_derivative(Real.exp, exp_chain, n.suc, c) = c.exp
        x - Real.0 = x
        (x - Real.0).pow(n.suc) = x.pow(n.suc)
        taylor_remainder(Real.exp, exp_chain, Real.0, x, c, n) =
            (c.exp / from_nat[Real](n.suc.factorial)) * x.pow(n.suc)
        x.exp = partial(exp_term(x), n.suc) +
            (c.exp / from_nat[Real](n.suc.factorial)) * x.pow(n.suc)
        exists(c2: Real) {
            Real.0 < c2 and c2 < x and
            x.exp = partial(exp_term(x), n.suc) +
                (c2.exp / from_nat[Real](n.suc.factorial)) * x.pow(n.suc)
        }
    }
}

/// Taylor's theorem for Real.exp at zero, with the remainder bounded: the interior
/// point c can be chosen so that c.exp < x.exp, bounding the Lagrange
/// remainder of order n by x.exp x^(n + 1) / (n + 1)!.
theorem taylor_exp_remainder_bound(n: Nat, x: Real) {
    Real.0 < x
    implies exists(c: Real) {
        Real.0 < c and c < x and c.exp < x.exp and
        x.exp = partial(exp_term(x), n.suc) +
            (c.exp / from_nat[Real](n.suc.factorial)) * x.pow(n.suc)
    }
} by {
    if Real.0 < x {
        taylor_exp_remainder(n, x)
        let c: Real satisfy {
            Real.0 < c and c < x and
            x.exp = partial(exp_term(x), n.suc) +
                (c.exp / from_nat[Real](n.suc.factorial)) * x.pow(n.suc)
        }
        Real.0 < c and c < x
        x.exp = partial(exp_term(x), n.suc) +
            (c.exp / from_nat[Real](n.suc.factorial)) * x.pow(n.suc)
        exp_increasing(c, x)
        c < x implies c.exp < x.exp
        c.exp < x.exp
        exists(c2: Real) {
            Real.0 < c2 and c2 < x and c2.exp < x.exp and
            x.exp = partial(exp_term(x), n.suc) +
                (c2.exp / from_nat[Real](n.suc.factorial)) * x.pow(n.suc)
        }
    }
}

