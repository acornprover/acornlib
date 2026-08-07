from data.basic.relation_basic import is_reflexive, is_transitive, is_antisymmetric, reflexive_self, transitive_step, antisymmetric_eq
from lte import LTE

/// A partial order is a relation that is reflexive, transitive, and antisymmetric.
/// Not all elements need to be comparable.
typeclass P: PartialOrder extends LTE {
    /// The order relation must be reflexive: every element is `≤` itself.
    reflexive {
        is_reflexive(P.lte)
    }

    /// The order relation must be transitive: if `a ≤ b` and `b ≤ c`, then `a ≤ c`.
    transitive {
        is_transitive(P.lte)
    }

    /// The order relation must be antisymmetric: if `a ≤ b` and `b ≤ a`, then `a = b`.
    antisymmetric {
        is_antisymmetric(P.lte)
    }
}

attributes P: PartialOrder {
    /// Strict less-than comparison.
    define lt(self, other: P) -> Bool {
        self <= other and self != other
    }

    /// Greater-than-or-equal-to comparison.
    define gte(self, other: P) -> Bool {
        other <= self
    }

    /// Strict greater-than comparison.
    define gt(self, other: P) -> Bool {
        other < self
    }
}

theorem lte_refl[P: PartialOrder](a: P) {
    a <= a
} by {
    reflexive_self(P.lte, a)
}

theorem gte_refl[P: PartialOrder](a: P) {
    a >= a
} by {
    lte_refl(a)
}

/// Every element is below itself.
theorem lte_ref[P: PartialOrder](a: P) {
    a <= a
} by {
    lte_refl(a)
}

/// Every element is below itself.
theorem lte_self[P: PartialOrder](a: P) {
    a <= a
} by {
    lte_refl(a)
}

/// Every element is above itself.
theorem gte_self[P: PartialOrder](a: P) {
    a >= a
} by {
    gte_refl(a)
}

/// Every element is below itself.
theorem le_refl[P: PartialOrder](a: P) {
    a <= a
} by {
    lte_refl(a)
}

/// Every element is above itself.
theorem ge_refl[P: PartialOrder](a: P) {
    a >= a
} by {
    gte_refl(a)
}

theorem lte_trans[P: PartialOrder](a: P, b: P, c: P) {
    a <= b and b <= c implies a <= c
} by {
    if a <= b and b <= c {
        transitive_step(P.lte, a, b, c)
        a <= c
    }
}

theorem gte_trans[P: PartialOrder](a: P, b: P, c: P) {
    a >= b and b >= c implies a >= c
} by {
    if a >= b and b >= c {
        c <= b
        b <= a
        lte_trans(c, b, a)
        c <= a
        a >= c
    }
}

/// The non-strict order is transitive.
theorem le_trans[P: PartialOrder](a: P, b: P, c: P) {
    a <= b and b <= c implies a <= c
} by {
    if a <= b and b <= c {
        lte_trans(a, b, c)
        a <= c
    }
}

/// The reverse non-strict order is transitive.
theorem ge_trans[P: PartialOrder](a: P, b: P, c: P) {
    a >= b and b >= c implies a >= c
} by {
    if a >= b and b >= c {
        gte_trans(a, b, c)
        a >= c
    }
}

theorem lte_antisymm[P: PartialOrder](a: P, b: P) {
    a <= b and b <= a implies a = b
} by {
    if a <= b and b <= a {
        antisymmetric_eq(P.lte, a, b)
        a = b
    }
}

/// Mutual non-strict bounds force equality.
theorem lte_both_ways_imp_eq[P: PartialOrder](a: P, b: P) {
    a <= b and b <= a implies a = b
} by {
    if a <= b and b <= a {
        lte_antisymm(a, b)
        a = b
    }
}

/// Mutual non-strict bounds force equality.
theorem le_antisymm[P: PartialOrder](a: P, b: P) {
    a <= b and b <= a implies a = b
} by {
    if a <= b and b <= a {
        lte_antisymm(a, b)
        a = b
    }
}

/// Mutual `<=` forces equality on a partial order.
theorem lte_and_gte_imp_eq[P: PartialOrder](a: P, b: P) {
    a <= b and a >= b implies a = b
} by {
    if a <= b and a >= b {
        b <= a
        lte_antisymm(a, b)
    }
}

/// A lower and an upper bound force equality.
theorem eq_of_le_of_ge[P: PartialOrder](a: P, b: P) {
    a <= b and a >= b implies a = b
} by {
    if a <= b and a >= b {
        lte_and_gte_imp_eq(a, b)
        a = b
    }
}

/// An upper and a lower bound force equality.
theorem eq_of_ge_of_le[P: PartialOrder](a: P, b: P) {
    a >= b and a <= b implies a = b
} by {
    if a >= b and a <= b {
        lte_and_gte_imp_eq(a, b)
        a = b
    }
}

/// Two partial-order elements are equal when their `<=`-profile agrees on every test point.
theorem eq_of_forall_lte_iff[P: PartialOrder](a: P, b: P) {
    (forall(c: P) { a <= c = b <= c }) implies a = b
} by {
    if forall(c: P) { a <= c = b <= c } {
        a <= a = b <= a
        lte_refl(a)
        b <= a
        a <= b = b <= b
        lte_refl(b)
        a <= b
        lte_antisymm(a, b)
        a = b
    }
}

/// Two partial-order elements are equal when their reverse `<=`-profile agrees on every test point.
theorem eq_of_forall_lte_iff_swap[P: PartialOrder](a: P, b: P) {
    (forall(c: P) { c <= a = c <= b }) implies a = b
} by {
    if forall(c: P) { c <= a = c <= b } {
        a <= a = a <= b
        lte_refl(a)
        a <= b
        b <= a = b <= b
        lte_refl(b)
        b <= a
        lte_antisymm(a, b)
        a = b
    }
}

theorem eq_imp_lte[P: PartialOrder](a: P, b: P) {
    a = b implies a <= b
} by {
    if a = b {
        lte_refl(a)
        a <= b
    }
}

theorem eq_imp_gte[P: PartialOrder](a: P, b: P) {
    a = b implies a >= b
} by {
    if a = b {
        gte_refl(a)
        a >= b
    }
}

/// Equality gives the corresponding non-strict lower bound.
theorem lte_of_eq[P: PartialOrder](a: P, b: P) {
    a = b implies a <= b
} by {
    if a = b {
        eq_imp_lte(a, b)
        a <= b
    }
}

/// Equality gives the corresponding non-strict lower bound.
theorem le_of_eq[P: PartialOrder](a: P, b: P) {
    a = b implies a <= b
} by {
    if a = b {
        lte_of_eq(a, b)
        a <= b
    }
}

/// Equality gives the corresponding non-strict upper bound.
theorem gte_of_eq[P: PartialOrder](a: P, b: P) {
    a = b implies a >= b
} by {
    if a = b {
        eq_imp_gte(a, b)
        a >= b
    }
}

/// Equality gives the corresponding non-strict upper bound.
theorem ge_of_eq[P: PartialOrder](a: P, b: P) {
    a = b implies a >= b
} by {
    if a = b {
        gte_of_eq(a, b)
        a >= b
    }
}

/// Two elements are equal exactly when they bound each other.
theorem eq_iff_lte_and_gte[P: PartialOrder](a: P, b: P) {
    a = b = (a <= b and a >= b)
} by {
    if a = b {
        eq_imp_lte(a, b)
        a <= b
        eq_imp_gte(a, b)
        a >= b
        a <= b and a >= b
    }
    if a <= b and a >= b {
        b <= a
        lte_antisymm(a, b)
        a = b
    }
    a = b = (a <= b and a >= b)
}

/// Two elements are equal exactly when they bound each other in reverse order.
theorem eq_iff_gte_and_lte[P: PartialOrder](a: P, b: P) {
    a = b = (a >= b and a <= b)
} by {
    if a = b {
        eq_imp_gte(a, b)
        a >= b
        eq_imp_lte(a, b)
        a <= b
        a >= b and a <= b
    }
    if a >= b and a <= b {
        b <= a
        lte_antisymm(a, b)
        a = b
    }
    a = b = (a >= b and a <= b)
}

/// Distinct elements cannot bound each other.
theorem ne_imp_not_lte_or_not_gte[P: PartialOrder](a: P, b: P) {
    a != b implies not (a <= b) or not (a >= b)
} by {
    if a != b {
        if a <= b {
            if a >= b {
                b <= a
                lte_antisymm(a, b)
                a = b
                false
            } else {
                not (a >= b)
                not (a <= b) or not (a >= b)
            }
        } else {
            not (a <= b)
            not (a <= b) or not (a >= b)
        }
    }
}

/// Distinct elements cannot bound each other, with the disjunction reversed.
theorem ne_imp_not_gte_or_not_lte[P: PartialOrder](a: P, b: P) {
    a != b implies not (a >= b) or not (a <= b)
} by {
    if a != b {
        ne_imp_not_lte_or_not_gte(a, b)
        if not (a <= b) {
            not (a >= b) or not (a <= b)
        } else {
            not (a >= b)
            not (a >= b) or not (a <= b)
        }
    }
}

/// Disequality is equivalent to the failure of at least one comparison.
theorem ne_iff_not_lte_or_not_gte[P: PartialOrder](a: P, b: P) {
    a != b = (not (a <= b) or not (a >= b))
} by {
    if a != b {
        ne_imp_not_lte_or_not_gte(a, b)
        not (a <= b) or not (a >= b)
    }
    if not (a <= b) or not (a >= b) {
        if a = b {
            eq_imp_lte(a, b)
            a <= b
            eq_imp_gte(a, b)
            a >= b
            false
        }
        a != b
    }
    a != b = (not (a <= b) or not (a >= b))
}

/// Disequality is equivalent to the reversed failure disjunction.
theorem ne_iff_not_gte_or_not_lte[P: PartialOrder](a: P, b: P) {
    a != b = (not (a >= b) or not (a <= b))
} by {
    if a != b {
        ne_imp_not_gte_or_not_lte(a, b)
        not (a >= b) or not (a <= b)
    }
    if not (a >= b) or not (a <= b) {
        if a = b {
            eq_imp_gte(a, b)
            a >= b
            eq_imp_lte(a, b)
            a <= b
            false
        }
        a != b
    }
    a != b = (not (a >= b) or not (a <= b))
}

theorem lt_imp_lte[P: PartialOrder](a: P, b: P) {
    a < b implies a <= b
} by {
    if a < b {
        a <= b
    }
}

/// Strict comparison gives the corresponding non-strict comparison.
theorem lte_of_lt[P: PartialOrder](a: P, b: P) {
    a < b implies a <= b
} by {
    if a < b {
        lt_imp_lte(a, b)
        a <= b
    }
}

/// Strict comparison gives the corresponding non-strict comparison.
theorem le_of_lt[P: PartialOrder](a: P, b: P) {
    a < b implies a <= b
} by {
    if a < b {
        lte_of_lt(a, b)
        a <= b
    }
}

theorem gt_imp_gte[P: PartialOrder](a: P, b: P) {
    a > b implies a >= b
} by {
    if a > b {
        a >= b
    }
}

/// Strict reverse comparison gives the corresponding non-strict reverse comparison.
theorem gte_of_gt[P: PartialOrder](a: P, b: P) {
    a > b implies a >= b
} by {
    if a > b {
        gt_imp_gte(a, b)
        a >= b
    }
}

/// Strict reverse comparison gives the corresponding non-strict reverse comparison.
theorem ge_of_gt[P: PartialOrder](a: P, b: P) {
    a > b implies a >= b
} by {
    if a > b {
        gte_of_gt(a, b)
        a >= b
    }
}

theorem lt_imp_ne[P: PartialOrder](a: P, b: P) {
    a < b implies a != b
} by {
    if a < b {
        a != b
    }
}

/// Strict comparison gives disequality.
theorem ne_of_lt[P: PartialOrder](a: P, b: P) {
    a < b implies a != b
} by {
    if a < b {
        lt_imp_ne(a, b)
        a != b
    }
}

/// Strict comparison gives inequality.
theorem not_eq_of_lt[P: PartialOrder](a: P, b: P) {
    a < b implies a != b
} by {
    if a < b {
        ne_of_lt(a, b)
        a != b
    }
}

/// Strict comparison gives disequality.
theorem lt_ne[P: PartialOrder](a: P, b: P) {
    a < b implies a != b
} by {
    if a < b {
        ne_of_lt(a, b)
        a != b
    }
}

theorem gt_imp_ne[P: PartialOrder](a: P, b: P) {
    a > b implies a != b
} by {
    if a > b {
        a != b
    }
}

/// Strict reverse comparison gives disequality.
theorem ne_of_gt[P: PartialOrder](a: P, b: P) {
    a > b implies a != b
} by {
    if a > b {
        gt_imp_ne(a, b)
        a != b
    }
}

/// Strict reverse comparison gives inequality.
theorem not_eq_of_gt[P: PartialOrder](a: P, b: P) {
    a > b implies a != b
} by {
    if a > b {
        ne_of_gt(a, b)
        a != b
    }
}

/// Strict reverse comparison gives disequality.
theorem gt_ne[P: PartialOrder](a: P, b: P) {
    a > b implies a != b
} by {
    if a > b {
        ne_of_gt(a, b)
        a != b
    }
}

theorem lt_imp_ne_symm[P: PartialOrder](a: P, b: P) {
    a < b implies b != a
} by {
    if a < b {
        a != b
        b != a
    }
}

/// Strict comparison gives disequality in the reverse order.
theorem ne_symm_of_lt[P: PartialOrder](a: P, b: P) {
    a < b implies b != a
} by {
    if a < b {
        lt_imp_ne_symm(a, b)
        b != a
    }
}

/// Strict comparison gives inequality in the reverse order.
theorem not_eq_symm_of_lt[P: PartialOrder](a: P, b: P) {
    a < b implies b != a
} by {
    if a < b {
        ne_symm_of_lt(a, b)
        b != a
    }
}

theorem gt_imp_ne_symm[P: PartialOrder](a: P, b: P) {
    a > b implies b != a
} by {
    if a > b {
        b < a
        b != a
    }
}

/// Strict reverse comparison gives disequality in the reverse order.
theorem ne_symm_of_gt[P: PartialOrder](a: P, b: P) {
    a > b implies b != a
} by {
    if a > b {
        gt_imp_ne_symm(a, b)
        b != a
    }
}

/// Strict reverse comparison gives inequality in the reverse order.
theorem not_eq_symm_of_gt[P: PartialOrder](a: P, b: P) {
    a > b implies b != a
} by {
    if a > b {
        ne_symm_of_gt(a, b)
        b != a
    }
}

theorem not_lt_ref[P: PartialOrder](a: P) {
    not (a < a)
} by {
    if a < a {
        a != a
        false
    }
}

/// No element is strictly below itself.
theorem lt_not_ref[P: PartialOrder](a: P) {
    not (a < a)
} by {
    not_lt_ref(a)
}

/// No element is strictly below itself.
theorem lt_irrefl[P: PartialOrder](a: P) {
    not (a < a)
} by {
    not_lt_ref(a)
}

theorem not_gt_ref[P: PartialOrder](a: P) {
    not (a > a)
} by {
    if a > a {
        a != a
        false
    }
}

/// No element is strictly above itself.
theorem gt_irrefl[P: PartialOrder](a: P) {
    not (a > a)
} by {
    not_gt_ref(a)
}

/// No element is strictly below itself.
theorem not_lt_self[P: PartialOrder](a: P) {
    not (a < a)
} by {
    not_lt_ref(a)
}

/// No element is strictly above itself.
theorem not_gt_self[P: PartialOrder](a: P) {
    not (a > a)
} by {
    not_gt_ref(a)
}

theorem lte_imp_not_gt[P: PartialOrder](a: P, b: P) {
    a <= b implies not (a > b)
} by {
    if a <= b {
        if a > b {
            b <= a
            lte_antisymm(a, b)
            a = b
            a != b
            false
        }
    }
}

/// A non-strict comparison rules out the opposite strict comparison.
theorem not_gt_of_lte[P: PartialOrder](a: P, b: P) {
    a <= b implies not (a > b)
} by {
    if a <= b {
        lte_imp_not_gt(a, b)
        not (a > b)
    }
}

/// A non-strict comparison rules out the opposite strict comparison.
theorem not_gt_of_le[P: PartialOrder](a: P, b: P) {
    a <= b implies not (a > b)
} by {
    if a <= b {
        not_gt_of_lte(a, b)
        not (a > b)
    }
}

theorem gte_imp_not_lt[P: PartialOrder](a: P, b: P) {
    a >= b implies not (a < b)
} by {
    if a >= b {
        if a < b {
            b <= a
            lte_antisymm(a, b)
            a = b
            a != b
            false
        }
    }
}

/// A non-strict reverse comparison rules out the strict comparison.
theorem not_lt_of_gte[P: PartialOrder](a: P, b: P) {
    a >= b implies not (a < b)
} by {
    if a >= b {
        gte_imp_not_lt(a, b)
        not (a < b)
    }
}

/// A non-strict reverse comparison rules out the strict comparison.
theorem not_lt_of_ge[P: PartialOrder](a: P, b: P) {
    a >= b implies not (a < b)
} by {
    if a >= b {
        not_lt_of_gte(a, b)
        not (a < b)
    }
}

/// A non-strict comparison rules out the swapped strict comparison.
theorem not_lt_swap_of_lte[P: PartialOrder](a: P, b: P) {
    a <= b implies not (b < a)
} by {
    if a <= b {
        not_gt_of_lte(a, b)
        not (a > b)
        not (b < a)
    }
}

/// A non-strict comparison rules out the swapped strict comparison.
theorem not_lt_swap_of_le[P: PartialOrder](a: P, b: P) {
    a <= b implies not (b < a)
} by {
    if a <= b {
        not_lt_swap_of_lte(a, b)
        not (b < a)
    }
}

/// A non-strict comparison rules out the swapped strict comparison.
theorem lte_imp_not_lt_swap[P: PartialOrder](a: P, b: P) {
    a <= b implies not (b < a)
} by {
    if a <= b {
        not_lt_swap_of_lte(a, b)
        not (b < a)
    }
}

/// A non-strict comparison rules out the swapped strict comparison.
theorem lte_imp_not_lt[P: PartialOrder](a: P, b: P) {
    a <= b implies not (b < a)
} by {
    if a <= b {
        not_lt_swap_of_lte(a, b)
        not (b < a)
    }
}

/// A non-strict comparison rules out the swapped strict comparison.
theorem le_imp_not_lt_swap[P: PartialOrder](a: P, b: P) {
    a <= b implies not (b < a)
} by {
    if a <= b {
        not_lt_swap_of_lte(a, b)
        not (b < a)
    }
}

/// `a > b` rules out `a <= b`.
theorem gt_imp_not_lte[P: PartialOrder](a: P, b: P) {
    a > b implies not (a <= b)
} by {
    if a > b {
        if a <= b {
            b < a
            b <= a
            lte_antisymm(a, b)
            a = b
            b != a
            false
        }
    }
}

/// A strict reverse comparison rules out the non-strict comparison.
theorem not_lte_of_gt[P: PartialOrder](a: P, b: P) {
    a > b implies not (a <= b)
} by {
    if a > b {
        gt_imp_not_lte(a, b)
        not (a <= b)
    }
}

/// A strict reverse comparison rules out the non-strict comparison.
theorem not_le_of_gt[P: PartialOrder](a: P, b: P) {
    a > b implies not (a <= b)
} by {
    if a > b {
        not_lte_of_gt(a, b)
        not (a <= b)
    }
}

/// `a < b` rules out `a >= b`.
theorem lt_imp_not_gte[P: PartialOrder](a: P, b: P) {
    a < b implies not (a >= b)
} by {
    if a < b {
        if a >= b {
            b <= a
            a <= b
            lte_antisymm(a, b)
            a = b
            a != b
            false
        }
    }
}

/// A strict comparison rules out the non-strict reverse comparison.
theorem not_gte_of_lt[P: PartialOrder](a: P, b: P) {
    a < b implies not (a >= b)
} by {
    if a < b {
        lt_imp_not_gte(a, b)
        not (a >= b)
    }
}

/// A strict comparison rules out the non-strict reverse comparison.
theorem not_ge_of_lt[P: PartialOrder](a: P, b: P) {
    a < b implies not (a >= b)
} by {
    if a < b {
        not_gte_of_lt(a, b)
        not (a >= b)
    }
}

/// A strict comparison rules out the swapped strict comparison.
theorem not_lt_both_ways[P: PartialOrder](a: P, b: P) {
    a < b implies not (b < a)
} by {
    if a < b {
        if b < a {
            a <= b
            b <= a
            lte_antisymm(a, b)
            a = b
            a != b
            false
        }
    }
}

/// A strict comparison rules out the swapped strict comparison.
theorem lt_not_symm[P: PartialOrder](a: P, b: P) {
    a < b implies not (b < a)
} by {
    if a < b {
        not_lt_both_ways(a, b)
        not (b < a)
    }
}

theorem lt_trans[P: PartialOrder](a: P, b: P, c: P) {
    a < b and b < c implies a < c
} by {
    if a < b and b < c {
        a <= b
        b <= c
        lte_trans(a, b, c)
        a <= c
        if a = c {
            c <= b
            b <= a
            lte_antisymm(a, b)
            a = b
            a != b
            false
        }
        a != c
        a < c
    }
}

theorem gt_trans[P: PartialOrder](a: P, b: P, c: P) {
    a > b and b > c implies a > c
} by {
    if a > b and b > c {
        c < b
        b < a
        lt_trans(c, b, a)
        c < a
        a > c
    }
}

theorem lt_of_lt_of_lte[P: PartialOrder](a: P, b: P, c: P) {
    a < b and b <= c implies a < c
} by {
    if a < b and b <= c {
        a <= b
        lte_trans(a, b, c)
        a <= c
        if a = c {
            c <= b
            b <= a
            lte_antisymm(a, b)
            a = b
            a != b
            false
        }
        a != c
        a < c
    }
}

theorem lt_of_lte_of_lt[P: PartialOrder](a: P, b: P, c: P) {
    a <= b and b < c implies a < c
} by {
    if a <= b and b < c {
        b <= c
        lte_trans(a, b, c)
        a <= c
        if a = c {
            c <= b
            b <= a
            lte_antisymm(a, b)
            a = b
            b != c
            false
        }
        a != c
        a < c
    }
}

theorem gt_of_gt_of_gte[P: PartialOrder](a: P, b: P, c: P) {
    a > b and b >= c implies a > c
} by {
    if a > b and b >= c {
        lt_of_lte_of_lt(c, b, a)
        c < a
        a > c
    }
}

theorem gt_of_gte_of_gt[P: PartialOrder](a: P, b: P, c: P) {
    a >= b and b > c implies a > c
} by {
    if a >= b and b > c {
        lt_of_lt_of_lte(c, b, a)
        c < a
        a > c
    }
}

/// A strict comparison followed by a non-strict comparison gives a strict comparison.
theorem lt_and_lte[P: PartialOrder](a: P, b: P, c: P) {
    a < b and b <= c implies a < c
} by {
    if a < b and b <= c {
        lt_of_lt_of_lte(a, b, c)
        a < c
    }
}

/// A non-strict comparison followed by a strict comparison gives a strict comparison.
theorem lte_and_lt[P: PartialOrder](a: P, b: P, c: P) {
    a <= b and b < c implies a < c
} by {
    if a <= b and b < c {
        lt_of_lte_of_lt(a, b, c)
        a < c
    }
}

/// A strict comparison followed by a non-strict comparison gives a strict comparison.
theorem lt_lte_trans[P: PartialOrder](a: P, b: P, c: P) {
    a < b and b <= c implies a < c
} by {
    if a < b and b <= c {
        lt_of_lt_of_lte(a, b, c)
        a < c
    }
}

/// A non-strict comparison followed by a strict comparison gives a strict comparison.
theorem lte_lt_trans[P: PartialOrder](a: P, b: P, c: P) {
    a <= b and b < c implies a < c
} by {
    if a <= b and b < c {
        lt_of_lte_of_lt(a, b, c)
        a < c
    }
}

/// A strict reverse comparison followed by a non-strict reverse comparison gives a strict reverse comparison.
theorem gt_and_gte[P: PartialOrder](a: P, b: P, c: P) {
    a > b and b >= c implies a > c
} by {
    if a > b and b >= c {
        gt_of_gt_of_gte(a, b, c)
        a > c
    }
}

/// A non-strict reverse comparison followed by a strict reverse comparison gives a strict reverse comparison.
theorem gte_and_gt[P: PartialOrder](a: P, b: P, c: P) {
    a >= b and b > c implies a > c
} by {
    if a >= b and b > c {
        gt_of_gte_of_gt(a, b, c)
        a > c
    }
}

/// A strict reverse comparison followed by a non-strict reverse comparison gives a strict reverse comparison.
theorem gt_gte_trans[P: PartialOrder](a: P, b: P, c: P) {
    a > b and b >= c implies a > c
} by {
    if a > b and b >= c {
        gt_of_gt_of_gte(a, b, c)
        a > c
    }
}

/// A non-strict reverse comparison followed by a strict reverse comparison gives a strict reverse comparison.
theorem gte_gt_trans[P: PartialOrder](a: P, b: P, c: P) {
    a >= b and b > c implies a > c
} by {
    if a >= b and b > c {
        gt_of_gte_of_gt(a, b, c)
        a > c
    }
}

/// A strict lower bound followed by a non-strict lower bound gives a non-strict lower bound.
theorem lte_of_lt_of_lte[P: PartialOrder](a: P, b: P, c: P) {
    a < b and b <= c implies a <= c
} by {
    if a < b and b <= c {
        lt_of_lt_of_lte(a, b, c)
        a < c
        lt_imp_lte(a, c)
        a <= c
    }
}

/// A non-strict lower bound followed by a strict lower bound gives a non-strict lower bound.
theorem lte_of_lte_of_lt[P: PartialOrder](a: P, b: P, c: P) {
    a <= b and b < c implies a <= c
} by {
    if a <= b and b < c {
        lt_of_lte_of_lt(a, b, c)
        a < c
        lt_imp_lte(a, c)
        a <= c
    }
}

/// A strict upper bound followed by a non-strict upper bound gives a non-strict upper bound.
theorem gte_of_gt_of_gte[P: PartialOrder](a: P, b: P, c: P) {
    a > b and b >= c implies a >= c
} by {
    if a > b and b >= c {
        gt_of_gt_of_gte(a, b, c)
        a > c
        gt_imp_gte(a, c)
        a >= c
    }
}

/// A non-strict upper bound followed by a strict upper bound gives a non-strict upper bound.
theorem gte_of_gte_of_gt[P: PartialOrder](a: P, b: P, c: P) {
    a >= b and b > c implies a >= c
} by {
    if a >= b and b > c {
        gt_of_gte_of_gt(a, b, c)
        a > c
        gt_imp_gte(a, c)
        a >= c
    }
}

/// A strict lower bound followed by a non-strict lower bound gives a non-strict lower bound.
theorem le_of_lt_of_lte[P: PartialOrder](a: P, b: P, c: P) {
    a < b and b <= c implies a <= c
} by {
    if a < b and b <= c {
        lte_of_lt_of_lte(a, b, c)
        a <= c
    }
}

/// A non-strict lower bound followed by a strict lower bound gives a non-strict lower bound.
theorem le_of_lte_of_lt[P: PartialOrder](a: P, b: P, c: P) {
    a <= b and b < c implies a <= c
} by {
    if a <= b and b < c {
        lte_of_lte_of_lt(a, b, c)
        a <= c
    }
}

/// A strict lower bound followed by a non-strict lower bound gives a strict lower bound.
theorem lt_of_lt_of_le[P: PartialOrder](a: P, b: P, c: P) {
    a < b and b <= c implies a < c
} by {
    if a < b and b <= c {
        lt_of_lt_of_lte(a, b, c)
        a < c
    }
}

/// A non-strict lower bound followed by a strict lower bound gives a strict lower bound.
theorem lt_of_le_of_lt[P: PartialOrder](a: P, b: P, c: P) {
    a <= b and b < c implies a < c
} by {
    if a <= b and b < c {
        lt_of_lte_of_lt(a, b, c)
        a < c
    }
}

/// A strict upper bound followed by a non-strict upper bound gives a non-strict upper bound.
theorem ge_of_gt_of_gte[P: PartialOrder](a: P, b: P, c: P) {
    a > b and b >= c implies a >= c
} by {
    if a > b and b >= c {
        gte_of_gt_of_gte(a, b, c)
        a >= c
    }
}

/// A non-strict upper bound followed by a strict upper bound gives a non-strict upper bound.
theorem ge_of_gte_of_gt[P: PartialOrder](a: P, b: P, c: P) {
    a >= b and b > c implies a >= c
} by {
    if a >= b and b > c {
        gte_of_gte_of_gt(a, b, c)
        a >= c
    }
}

/// A strict upper bound followed by a non-strict upper bound gives a strict upper bound.
theorem gt_of_gt_of_ge[P: PartialOrder](a: P, b: P, c: P) {
    a > b and b >= c implies a > c
} by {
    if a > b and b >= c {
        gt_of_gt_of_gte(a, b, c)
        a > c
    }
}

/// A non-strict upper bound followed by a strict upper bound gives a strict upper bound.
theorem gt_of_ge_of_gt[P: PartialOrder](a: P, b: P, c: P) {
    a >= b and b > c implies a > c
} by {
    if a >= b and b > c {
        gt_of_gte_of_gt(a, b, c)
        a > c
    }
}

/// Strict order is asymmetric: `a < b` rules out `b < a`.
theorem lt_asymm[P: PartialOrder](a: P, b: P) {
    a < b implies not (b < a)
} by {
    if a < b {
        if b < a {
            lt_trans(a, b, a)
            a < a
            a != a
            false
        }
    }
}

/// Strict greater-than is asymmetric: `a > b` rules out `b > a`.
theorem gt_asymm[P: PartialOrder](a: P, b: P) {
    a > b implies not (b > a)
} by {
    if a > b {
        b < a
        lt_asymm(b, a)
        not (a < b)
        not (b > a)
    }
}

/// A strict comparison rules out the reverse strict comparison.
theorem not_gt_of_lt[P: PartialOrder](a: P, b: P) {
    a < b implies not (a > b)
} by {
    if a < b {
        if a > b {
            b < a
            lt_asymm(a, b)
            false
        }
    }
}

/// A strict reverse comparison rules out the strict comparison.
theorem not_lt_of_gt[P: PartialOrder](a: P, b: P) {
    a > b implies not (a < b)
} by {
    if a > b {
        if a < b {
            b < a
            lt_asymm(b, a)
            false
        }
    }
}

/// A strict comparison rules out the reverse strict comparison.
theorem lt_imp_not_gt[P: PartialOrder](a: P, b: P) {
    a < b implies not (a > b)
} by {
    if a < b {
        not_gt_of_lt(a, b)
        not (a > b)
    }
}

/// A strict reverse comparison rules out the strict comparison.
theorem gt_imp_not_lt[P: PartialOrder](a: P, b: P) {
    a > b implies not (a < b)
} by {
    if a > b {
        not_lt_of_gt(a, b)
        not (a < b)
    }
}

/// Replacing the right side of `<=` with an equal element preserves `<=`.
theorem lte_of_lte_of_eq[P: PartialOrder](a: P, b: P, c: P) {
    a <= b and b = c implies a <= c
} by {
    if a <= b and b = c {
        a <= c
    }
}

/// Transitivity with equality on the right.
theorem lte_trans_eq[P: PartialOrder](a: P, b: P, c: P) {
    a <= b and b = c implies a <= c
} by {
    if a <= b and b = c {
        lte_of_lte_of_eq(a, b, c)
        a <= c
    }
}

/// Replacing the left side of `<=` with an equal element preserves `<=`.
theorem lte_of_eq_of_lte[P: PartialOrder](a: P, b: P, c: P) {
    a = b and b <= c implies a <= c
} by {
    if a = b and b <= c {
        a <= c
    }
}

/// Transitivity with equality on the left.
theorem lte_eq_trans[P: PartialOrder](a: P, b: P, c: P) {
    a = b and b <= c implies a <= c
} by {
    if a = b and b <= c {
        lte_of_eq_of_lte(a, b, c)
        a <= c
    }
}

/// Replacing the right side of `<=` with an equal element preserves `<=`.
theorem le_of_le_of_eq[P: PartialOrder](a: P, b: P, c: P) {
    a <= b and b = c implies a <= c
} by {
    if a <= b and b = c {
        lte_of_lte_of_eq(a, b, c)
        a <= c
    }
}

/// Replacing the left side of `<=` with an equal element preserves `<=`.
theorem le_of_eq_of_le[P: PartialOrder](a: P, b: P, c: P) {
    a = b and b <= c implies a <= c
} by {
    if a = b and b <= c {
        lte_of_eq_of_lte(a, b, c)
        a <= c
    }
}

/// Replacing the right side of `<` with an equal element preserves `<`.
theorem lt_of_lt_of_eq[P: PartialOrder](a: P, b: P, c: P) {
    a < b and b = c implies a < c
} by {
    if a < b and b = c {
        a < c
    }
}

/// Replacing the left side of `<` with an equal element preserves `<`.
theorem lt_of_eq_of_lt[P: PartialOrder](a: P, b: P, c: P) {
    a = b and b < c implies a < c
} by {
    if a = b and b < c {
        a < c
    }
}

/// Replacing the right side of `>=` with an equal element preserves `>=`.
theorem gte_of_gte_of_eq[P: PartialOrder](a: P, b: P, c: P) {
    a >= b and b = c implies a >= c
} by {
    if a >= b and b = c {
        a >= c
    }
}

/// Replacing the left side of `>=` with an equal element preserves `>=`.
theorem gte_of_eq_of_gte[P: PartialOrder](a: P, b: P, c: P) {
    a = b and b >= c implies a >= c
} by {
    if a = b and b >= c {
        a >= c
    }
}

/// Replacing the right side of `>=` with an equal element preserves `>=`.
theorem ge_of_ge_of_eq[P: PartialOrder](a: P, b: P, c: P) {
    a >= b and b = c implies a >= c
} by {
    if a >= b and b = c {
        gte_of_gte_of_eq(a, b, c)
        a >= c
    }
}

/// Replacing the left side of `>=` with an equal element preserves `>=`.
theorem ge_of_eq_of_ge[P: PartialOrder](a: P, b: P, c: P) {
    a = b and b >= c implies a >= c
} by {
    if a = b and b >= c {
        gte_of_eq_of_gte(a, b, c)
        a >= c
    }
}

/// Replacing the right side of `>` with an equal element preserves `>`.
theorem gt_of_gt_of_eq[P: PartialOrder](a: P, b: P, c: P) {
    a > b and b = c implies a > c
} by {
    if a > b and b = c {
        a > c
    }
}

/// Replacing the left side of `>` with an equal element preserves `>`.
theorem gt_of_eq_of_gt[P: PartialOrder](a: P, b: P, c: P) {
    a = b and b > c implies a > c
} by {
    if a = b and b > c {
        a > c
    }
}

/// `<` decomposes into `<=` together with inequality.
theorem lt_iff_lte_and_ne[P: PartialOrder](a: P, b: P) {
    a < b = (a <= b and a != b)
}

/// The conjunction `a <= b` and `a != b` is exactly `a < b`.
theorem lte_and_ne_iff_lt[P: PartialOrder](a: P, b: P) {
    (a <= b and a != b) = (a < b)
} by {
    if a <= b and a != b {
        a < b
    }
    if a < b {
        a <= b
        a != b
        a <= b and a != b
    }
    (a <= b and a != b) = (a < b)
}

/// `>` decomposes into `>=` together with inequality.
theorem gt_iff_gte_and_ne[P: PartialOrder](a: P, b: P) {
    a > b = (a >= b and a != b)
} by {
    if a > b {
        b < a
        a != b
    }
    if a >= b and a != b {
        b <= a
        b != a
        b < a
        a > b
    }
    a > b = (a >= b and a != b)
}

/// The conjunction `a >= b` and `a != b` is exactly `a > b`.
theorem gte_and_ne_iff_gt[P: PartialOrder](a: P, b: P) {
    (a >= b and a != b) = (a > b)
} by {
    if a >= b and a != b {
        b <= a
        b != a
        b < a
        a > b
    }
    if a > b {
        a >= b
        a != b
        a >= b and a != b
    }
    (a >= b and a != b) = (a > b)
}

/// `<` is the asymmetric part of `<=`: `a < b` iff not `b <= a` and `a <= b`.
theorem lt_iff_lte_not_lte_swap[P: PartialOrder](a: P, b: P) {
    a < b = (not (b <= a) and a <= b)
} by {
    if a < b {
        a <= b
        if b <= a {
            lte_antisymm(a, b)
            a = b
            a != b
            false
        }
    }
    if not (b <= a) and a <= b {
        if a = b {
            b <= a
            false
        }
        a != b
        a < b
    }
    a < b = (not (b <= a) and a <= b)
}

/// `>` is the asymmetric part of `>=`: `a > b` iff not `a <= b` and `b <= a`.
theorem gt_iff_lte_swap_not_lte[P: PartialOrder](a: P, b: P) {
    a > b = (not (a <= b) and b <= a)
} by {
    if a > b {
        b <= a
        if a <= b {
            lte_antisymm(a, b)
            a = b
            b < a
            b != a
            false
        }
    }
    if not (a <= b) and b <= a {
        if a = b {
            a <= b
            false
        }
        a != b
        b != a
        b < a
        a > b
    }
    a > b = (not (a <= b) and b <= a)
}

/// `<=` splits into the disjunction of strict inequality and equality.
theorem lte_iff_lt_or_eq[P: PartialOrder](a: P, b: P) {
    a <= b = (a < b or a = b)
} by {
    if a <= b {
        if a = b {
            a < b or a = b
        } else {
            a != b
            a < b
            a < b or a = b
        }
    }
    if a < b or a = b {
        if a < b {
            lt_imp_lte(a, b)
            a <= b
        } else {
            a = b
            eq_imp_lte(a, b)
            a <= b
        }
    }
    a <= b = (a < b or a = b)
}

/// `<=` splits into the disjunction of equality and strict inequality.
theorem lte_iff_eq_or_lt[P: PartialOrder](a: P, b: P) {
    a <= b = (a = b or a < b)
} by {
    if a <= b {
        if a = b {
            a = b or a < b
        } else {
            a != b
            a < b
            a = b or a < b
        }
    }
    if a = b or a < b {
        if a = b {
            eq_imp_lte(a, b)
            a <= b
        } else {
            a < b
            lt_imp_lte(a, b)
            a <= b
        }
    }
    a <= b = (a = b or a < b)
}

/// `>=` splits into the disjunction of strict greater-than and equality.
theorem gte_iff_gt_or_eq[P: PartialOrder](a: P, b: P) {
    a >= b = (a > b or a = b)
} by {
    if a >= b {
        b <= a
        lte_iff_lt_or_eq(b, a)
        b < a or b = a
        if b < a {
            a > b
            a > b or a = b
        } else {
            b = a
            a = b
            a > b or a = b
        }
    }
    if a > b or a = b {
        if a > b {
            gt_imp_gte(a, b)
            a >= b
        } else {
            a = b
            eq_imp_gte(a, b)
            a >= b
        }
    }
    a >= b = (a > b or a = b)
}

/// `>=` splits into the disjunction of equality and strict greater-than.
theorem gte_iff_eq_or_gt[P: PartialOrder](a: P, b: P) {
    a >= b = (a = b or a > b)
} by {
    if a >= b {
        b <= a
        lte_iff_eq_or_lt(b, a)
        b = a or b < a
        if b = a {
            a = b
            a = b or a > b
        } else {
            b < a
            a > b
            a = b or a > b
        }
    }
    if a = b or a > b {
        if a = b {
            eq_imp_gte(a, b)
            a >= b
        } else {
            a > b
            gt_imp_gte(a, b)
            a >= b
        }
    }
    a >= b = (a = b or a > b)
}

/// `a <= b` together with `a != b` upgrades to `a < b`.
theorem lt_of_lte_of_ne[P: PartialOrder](a: P, b: P) {
    a <= b and a != b implies a < b
} by {
    if a <= b and a != b {
        a < b
    }
}

/// `a != b` together with `a <= b` upgrades to `a < b`.
theorem lt_of_ne_of_lte[P: PartialOrder](a: P, b: P) {
    a != b and a <= b implies a < b
} by {
    if a != b and a <= b {
        lt_of_lte_of_ne(a, b)
        a < b
    }
}

/// `a >= b` together with `a != b` upgrades to `a > b`.
theorem gt_of_gte_of_ne[P: PartialOrder](a: P, b: P) {
    a >= b and a != b implies a > b
} by {
    if a >= b and a != b {
        b <= a
        b != a
        b < a
        a > b
    }
}

/// `a != b` together with `a >= b` upgrades to `a > b`.
theorem gt_of_ne_of_gte[P: PartialOrder](a: P, b: P) {
    a != b and a >= b implies a > b
} by {
    if a != b and a >= b {
        gt_of_gte_of_ne(a, b)
        a > b
    }
}

/// `a <= b` together with `b != a` upgrades to `a < b`.
theorem lt_of_lte_of_ne_symm[P: PartialOrder](a: P, b: P) {
    a <= b and b != a implies a < b
} by {
    if a <= b and b != a {
        if a = b {
            false
        }
        a != b
        lt_of_lte_of_ne(a, b)
        a < b
    }
}

/// `b != a` together with `a <= b` upgrades to `a < b`.
theorem lt_of_ne_symm_of_lte[P: PartialOrder](a: P, b: P) {
    b != a and a <= b implies a < b
} by {
    if b != a and a <= b {
        lt_of_lte_of_ne_symm(a, b)
        a < b
    }
}

/// `a >= b` together with `b != a` upgrades to `a > b`.
theorem gt_of_gte_of_ne_symm[P: PartialOrder](a: P, b: P) {
    a >= b and b != a implies a > b
} by {
    if a >= b and b != a {
        if a = b {
            false
        }
        a != b
        gt_of_gte_of_ne(a, b)
        a > b
    }
}

/// `b != a` together with `a >= b` upgrades to `a > b`.
theorem gt_of_ne_symm_of_gte[P: PartialOrder](a: P, b: P) {
    b != a and a >= b implies a > b
} by {
    if b != a and a >= b {
        gt_of_gte_of_ne_symm(a, b)
        a > b
    }
}

/// On a partial order, `a <= b` either coincides with equality or is strict.
theorem eq_or_lt_of_lte[P: PartialOrder](a: P, b: P) {
    a <= b implies a = b or a < b
} by {
    if a <= b {
        if a = b {
            a = b or a < b
        } else {
            a < b
            a = b or a < b
        }
    }
}

/// On a partial order, `a <= b` is either strict or equality.
theorem lt_or_eq_of_lte[P: PartialOrder](a: P, b: P) {
    a <= b implies a < b or a = b
} by {
    if a <= b {
        eq_or_lt_of_lte(a, b)
        if a = b {
            a < b or a = b
        } else {
            a < b
            a < b or a = b
        }
    }
}

/// On a partial order, `a >= b` either coincides with equality or is strict.
theorem eq_or_gt_of_gte[P: PartialOrder](a: P, b: P) {
    a >= b implies a = b or a > b
} by {
    if a >= b {
        if a = b {
            a = b or a > b
        } else {
            b <= a
            b != a
            b < a
            a > b
            a = b or a > b
        }
    }
}

/// On a partial order, `a >= b` is either strict or equality.
theorem gt_or_eq_of_gte[P: PartialOrder](a: P, b: P) {
    a >= b implies a > b or a = b
} by {
    if a >= b {
        eq_or_gt_of_gte(a, b)
        if a = b {
            a > b or a = b
        } else {
            a > b
            a > b or a = b
        }
    }
}

/// `<` decomposes into `<=` together with reversed inequality.
theorem lt_iff_lte_and_ne_symm[P: PartialOrder](a: P, b: P) {
    a < b = (a <= b and b != a)
} by {
    if a < b {
        a <= b
        lt_imp_ne_symm(a, b)
        b != a
        a <= b and b != a
    }
    if a <= b and b != a {
        lt_of_lte_of_ne_symm(a, b)
        a < b
    }
    a < b = (a <= b and b != a)
}

/// `>` decomposes into `>=` together with reversed inequality.
theorem gt_iff_gte_and_ne_symm[P: PartialOrder](a: P, b: P) {
    a > b = (a >= b and b != a)
} by {
    if a > b {
        a >= b
        gt_imp_ne_symm(a, b)
        b != a
        a >= b and b != a
    }
    if a >= b and b != a {
        gt_of_gte_of_ne_symm(a, b)
        a > b
    }
    a > b = (a >= b and b != a)
}

/// `<` decomposes into `<=` together with inequality.
theorem lt_iff_le_and_ne[P: PartialOrder](a: P, b: P) {
    a < b = (a <= b and a != b)
} by {
    lt_iff_lte_and_ne(a, b)
}

/// The conjunction `a <= b` and `a != b` is exactly `a < b`.
theorem le_and_ne_iff_lt[P: PartialOrder](a: P, b: P) {
    (a <= b and a != b) = (a < b)
} by {
    lte_and_ne_iff_lt(a, b)
}

/// `>` decomposes into `>=` together with inequality.
theorem gt_iff_ge_and_ne[P: PartialOrder](a: P, b: P) {
    a > b = (a >= b and a != b)
} by {
    gt_iff_gte_and_ne(a, b)
}

/// The conjunction `a >= b` and `a != b` is exactly `a > b`.
theorem ge_and_ne_iff_gt[P: PartialOrder](a: P, b: P) {
    (a >= b and a != b) = (a > b)
} by {
    gte_and_ne_iff_gt(a, b)
}

/// `<` is the asymmetric part of `<=`: `a < b` iff not `b <= a` and `a <= b`.
theorem lt_iff_le_not_le_swap[P: PartialOrder](a: P, b: P) {
    a < b = (not (b <= a) and a <= b)
} by {
    lt_iff_lte_not_lte_swap(a, b)
}

/// `>` is the asymmetric part of `>=`: `a > b` iff not `a <= b` and `b <= a`.
theorem gt_iff_le_swap_not_le[P: PartialOrder](a: P, b: P) {
    a > b = (not (a <= b) and b <= a)
} by {
    gt_iff_lte_swap_not_lte(a, b)
}

/// `<=` splits into the disjunction of strict inequality and equality.
theorem le_iff_lt_or_eq[P: PartialOrder](a: P, b: P) {
    a <= b = (a < b or a = b)
} by {
    lte_iff_lt_or_eq(a, b)
}

/// `<=` splits into the disjunction of equality and strict inequality.
theorem le_iff_eq_or_lt[P: PartialOrder](a: P, b: P) {
    a <= b = (a = b or a < b)
} by {
    lte_iff_eq_or_lt(a, b)
}

/// `>=` splits into the disjunction of strict greater-than and equality.
theorem ge_iff_gt_or_eq[P: PartialOrder](a: P, b: P) {
    a >= b = (a > b or a = b)
} by {
    gte_iff_gt_or_eq(a, b)
}

/// `>=` splits into the disjunction of equality and strict greater-than.
theorem ge_iff_eq_or_gt[P: PartialOrder](a: P, b: P) {
    a >= b = (a = b or a > b)
} by {
    gte_iff_eq_or_gt(a, b)
}

/// `a <= b` together with `a != b` upgrades to `a < b`.
theorem lt_of_le_of_ne[P: PartialOrder](a: P, b: P) {
    a <= b and a != b implies a < b
} by {
    if a <= b and a != b {
        lt_of_lte_of_ne(a, b)
        a < b
    }
}

/// `a != b` together with `a <= b` upgrades to `a < b`.
theorem lt_of_ne_of_le[P: PartialOrder](a: P, b: P) {
    a != b and a <= b implies a < b
} by {
    if a != b and a <= b {
        lt_of_ne_of_lte(a, b)
        a < b
    }
}

/// `a >= b` together with `a != b` upgrades to `a > b`.
theorem gt_of_ge_of_ne[P: PartialOrder](a: P, b: P) {
    a >= b and a != b implies a > b
} by {
    if a >= b and a != b {
        gt_of_gte_of_ne(a, b)
        a > b
    }
}

/// `a != b` together with `a >= b` upgrades to `a > b`.
theorem gt_of_ne_of_ge[P: PartialOrder](a: P, b: P) {
    a != b and a >= b implies a > b
} by {
    if a != b and a >= b {
        gt_of_ne_of_gte(a, b)
        a > b
    }
}

/// `a <= b` together with `b != a` upgrades to `a < b`.
theorem lt_of_le_of_ne_symm[P: PartialOrder](a: P, b: P) {
    a <= b and b != a implies a < b
} by {
    if a <= b and b != a {
        lt_of_lte_of_ne_symm(a, b)
        a < b
    }
}

/// `b != a` together with `a <= b` upgrades to `a < b`.
theorem lt_of_ne_symm_of_le[P: PartialOrder](a: P, b: P) {
    b != a and a <= b implies a < b
} by {
    if b != a and a <= b {
        lt_of_ne_symm_of_lte(a, b)
        a < b
    }
}

/// `a >= b` together with `b != a` upgrades to `a > b`.
theorem gt_of_ge_of_ne_symm[P: PartialOrder](a: P, b: P) {
    a >= b and b != a implies a > b
} by {
    if a >= b and b != a {
        gt_of_gte_of_ne_symm(a, b)
        a > b
    }
}

/// `b != a` together with `a >= b` upgrades to `a > b`.
theorem gt_of_ne_symm_of_ge[P: PartialOrder](a: P, b: P) {
    b != a and a >= b implies a > b
} by {
    if b != a and a >= b {
        gt_of_ne_symm_of_gte(a, b)
        a > b
    }
}

/// On a partial order, `a <= b` either coincides with equality or is strict.
theorem eq_or_lt_of_le[P: PartialOrder](a: P, b: P) {
    a <= b implies a = b or a < b
} by {
    if a <= b {
        eq_or_lt_of_lte(a, b)
        a = b or a < b
    }
}

/// On a partial order, `a <= b` is either strict or equality.
theorem lt_or_eq_of_le[P: PartialOrder](a: P, b: P) {
    a <= b implies a < b or a = b
} by {
    if a <= b {
        lt_or_eq_of_lte(a, b)
        a < b or a = b
    }
}

/// On a partial order, `a >= b` either coincides with equality or is strict.
theorem eq_or_gt_of_ge[P: PartialOrder](a: P, b: P) {
    a >= b implies a = b or a > b
} by {
    if a >= b {
        eq_or_gt_of_gte(a, b)
        a = b or a > b
    }
}

/// On a partial order, `a >= b` is either strict or equality.
theorem gt_or_eq_of_ge[P: PartialOrder](a: P, b: P) {
    a >= b implies a > b or a = b
} by {
    if a >= b {
        gt_or_eq_of_gte(a, b)
        a > b or a = b
    }
}

/// `<` decomposes into `<=` together with reversed inequality.
theorem lt_iff_le_and_ne_symm[P: PartialOrder](a: P, b: P) {
    a < b = (a <= b and b != a)
} by {
    lt_iff_lte_and_ne_symm(a, b)
}

/// `>` decomposes into `>=` together with reversed inequality.
theorem gt_iff_ge_and_ne_symm[P: PartialOrder](a: P, b: P) {
    a > b = (a >= b and b != a)
} by {
    gt_iff_gte_and_ne_symm(a, b)
}

/// A linear order (total order) is a partial order where all elements are comparable.
typeclass L: LinearOrder extends PartialOrder {
    /// All elements are comparable: for any two elements `a` and `b`, either `a ≤ b` or `b ≤ a`.
    totality(a: L, b: L) {
        a <= b or b <= a
    }
}

attributes L: LinearOrder {
    /// Yields the smaller of two elements.
    define min(self, other: L) -> L {
        if self <= other {
            self
        } else {
            other
        }
    }

    /// Yields the larger of two elements.
    define max(self, other: L) -> L {
        if other <= self {
            self
        } else {
            other
        }
    }
}

theorem lte_or_gte[L: LinearOrder](a: L, b: L) {
    a <= b or a >= b
} by {
    a <= b or b <= a
}

/// Any two elements are comparable.
theorem lte_or_lte_swap[L: LinearOrder](a: L, b: L) {
    a <= b or b <= a
} by {
    lte_or_gte(a, b)
}

/// Any two elements are comparable.
theorem le_or_le_swap[L: LinearOrder](a: L, b: L) {
    a <= b or b <= a
} by {
    lte_or_lte_swap(a, b)
}

/// Any two elements are comparable.
theorem le_total[L: LinearOrder](a: L, b: L) {
    a <= b or b <= a
} by {
    lte_or_lte_swap(a, b)
}

/// Any two elements are comparable.
theorem linear_order_total[L: LinearOrder](a: L, b: L) {
    a <= b or b <= a
} by {
    lte_or_lte_swap(a, b)
}

/// Any two elements are comparable.
theorem le_or_ge[L: LinearOrder](a: L, b: L) {
    a <= b or a >= b
} by {
    lte_or_gte(a, b)
}

/// Any two elements are comparable in the reverse disjunction order.
theorem ge_or_le[L: LinearOrder](a: L, b: L) {
    a >= b or a <= b
} by {
    lte_or_gte(a, b)
    if a <= b {
        a >= b or a <= b
    } else {
        a >= b
        a >= b or a <= b
    }
}

/// Any two elements are comparable in reverse-order form.
theorem gte_or_gte_swap[L: LinearOrder](a: L, b: L) {
    a >= b or b >= a
} by {
    lte_or_lte_swap(b, a)
    if b <= a {
        a >= b
        a >= b or b >= a
    } else {
        a <= b
        b >= a
        a >= b or b >= a
    }
}

/// Any two elements are comparable in reverse-order form.
theorem ge_or_ge_swap[L: LinearOrder](a: L, b: L) {
    a >= b or b >= a
} by {
    gte_or_gte_swap(a, b)
}

/// Either `a < b` or `b <= a`.
theorem lt_or_lte[L: LinearOrder](a: L, b: L) {
    a < b or b <= a
} by {
    if b <= a {
        b <= a
        a < b or b <= a
    } else {
        a <= b or b <= a
        a <= b
        if a = b {
            b <= a
            false
        }
        a != b
        a < b
        a < b or b <= a
    }
}

/// Either `a < b` or `b <= a`.
theorem lt_or_le_swap[L: LinearOrder](a: L, b: L) {
    a < b or b <= a
} by {
    lt_or_lte(a, b)
}

/// Either `a <= b` or `b < a`.
theorem lte_or_lt[L: LinearOrder](a: L, b: L) {
    a <= b or b < a
} by {
    if a <= b {
        a <= b or b < a
    } else {
        a <= b or b <= a
        b <= a
        if b = a {
            a <= b
            false
        }
        b != a
        b < a
        a <= b or b < a
    }
}

/// Either `a <= b` or `b < a`.
theorem le_or_lt_swap[L: LinearOrder](a: L, b: L) {
    a <= b or b < a
} by {
    lte_or_lt(a, b)
}

theorem not_lte_imp_gt[L: LinearOrder](a: L, b: L) {
    not a <= b implies a > b
} by {
    if not a <= b {
        a <= b or b <= a
        b <= a
        if b = a {
            a <= b
            false
        }
        b < a
        a > b
    }
}

/// Failure of `<=` gives strict greater-than.
theorem gt_of_not_le[L: LinearOrder](a: L, b: L) {
    not (a <= b) implies a > b
} by {
    if not (a <= b) {
        not_lte_imp_gt(a, b)
        a > b
    }
}

theorem not_gte_imp_lt[L: LinearOrder](a: L, b: L) {
    not a >= b implies a < b
} by {
    if not a >= b {
        a <= b or b <= a
        a <= b
        if a = b {
            b <= a
            a >= b
            false
        }
        a < b
    }
}

/// Failure of `>=` gives strict less-than.
theorem lt_of_not_ge[L: LinearOrder](a: L, b: L) {
    not (a >= b) implies a < b
} by {
    if not (a >= b) {
        not_gte_imp_lt(a, b)
        a < b
    }
}

theorem not_lt_imp_gte[L: LinearOrder](a: L, b: L) {
    not a < b implies a >= b
} by {
    if not a < b {
        a <= b or b <= a
        if a <= b {
            if a = b {
                b <= a
                a >= b
            } else {
                a < b
                false
            }
        } else {
            b <= a
            a >= b
        }
    }
}

/// Failure of `<` gives non-strict greater-than.
theorem ge_of_not_lt[L: LinearOrder](a: L, b: L) {
    not (a < b) implies a >= b
} by {
    if not (a < b) {
        not_lt_imp_gte(a, b)
        a >= b
    }
}

theorem not_gt_imp_lte[L: LinearOrder](a: L, b: L) {
    not a > b implies a <= b
} by {
    if not a > b {
        a <= b or b <= a
        if a <= b {
            a <= b
        } else {
            b <= a
            if b = a {
                a <= b
            } else {
                b < a
                a > b
                false
            }
        }
    }
}

/// Failure of `>` gives non-strict less-than.
theorem le_of_not_gt[L: LinearOrder](a: L, b: L) {
    not (a > b) implies a <= b
} by {
    if not (a > b) {
        not_gt_imp_lte(a, b)
        a <= b
    }
}

/// Negation of `<=` is exactly `>` on a linear order.
theorem not_lte_iff_gt[L: LinearOrder](a: L, b: L) {
    (not a <= b) = (a > b)
} by {
    if not a <= b {
        not_lte_imp_gt(a, b)
    }
    if a > b {
        gt_imp_not_lte(a, b)
    }
}

/// Negation of `<=` is exactly `>` on a linear order.
theorem not_le_iff_gt[L: LinearOrder](a: L, b: L) {
    (not a <= b) = (a > b)
} by {
    not_lte_iff_gt(a, b)
}

/// Negation of `>=` is exactly `<` on a linear order.
theorem not_gte_iff_lt[L: LinearOrder](a: L, b: L) {
    (not a >= b) = (a < b)
} by {
    if not a >= b {
        not_gte_imp_lt(a, b)
    }
    if a < b {
        lt_imp_not_gte(a, b)
    }
}

/// Negation of `>=` is exactly `<` on a linear order.
theorem not_ge_iff_lt[L: LinearOrder](a: L, b: L) {
    (not a >= b) = (a < b)
} by {
    not_gte_iff_lt(a, b)
}

/// Negation of `<` is exactly `>=` on a linear order.
theorem not_lt_iff_gte[L: LinearOrder](a: L, b: L) {
    (not a < b) = (a >= b)
} by {
    if not a < b {
        not_lt_imp_gte(a, b)
    }
    if a >= b {
        gte_imp_not_lt(a, b)
    }
}

/// Negation of `<` is exactly `>=` on a linear order.
theorem not_lt_iff_ge[L: LinearOrder](a: L, b: L) {
    (not a < b) = (a >= b)
} by {
    not_lt_iff_gte(a, b)
}

/// Negation of `>` is exactly `<=` on a linear order.
theorem not_gt_iff_lte[L: LinearOrder](a: L, b: L) {
    (not a > b) = (a <= b)
} by {
    if not a > b {
        not_gt_imp_lte(a, b)
    }
    if a <= b {
        lte_imp_not_gt(a, b)
    }
}

/// Negation of `>` is exactly `<=` on a linear order.
theorem not_gt_iff_le[L: LinearOrder](a: L, b: L) {
    (not a > b) = (a <= b)
} by {
    not_gt_iff_lte(a, b)
}

theorem lte_or_gt[L: LinearOrder](a: L, b: L) {
    a <= b or a > b
} by {
    if a <= b {
        a <= b or a > b
    } else {
        not_lte_imp_gt(a, b)
        a > b
        a <= b or a > b
    }
}

/// Either `a <= b` or `a > b`.
theorem le_or_gt[L: LinearOrder](a: L, b: L) {
    a <= b or a > b
} by {
    lte_or_gt(a, b)
}

/// Either `a > b` or `a <= b`.
theorem gt_or_le[L: LinearOrder](a: L, b: L) {
    a > b or a <= b
} by {
    lte_or_gt(a, b)
    if a <= b {
        a > b or a <= b
    } else {
        a > b
        a > b or a <= b
    }
}

theorem gte_or_lt[L: LinearOrder](a: L, b: L) {
    a >= b or a < b
} by {
    if b <= a {
        a >= b or a < b
    } else {
        not_gte_imp_lt(a, b)
        a < b
        a >= b or a < b
    }
}

/// Either `a >= b` or `a < b`.
theorem ge_or_lt[L: LinearOrder](a: L, b: L) {
    a >= b or a < b
} by {
    gte_or_lt(a, b)
}

/// Either `a < b` or `a >= b`.
theorem lt_or_ge[L: LinearOrder](a: L, b: L) {
    a < b or a >= b
} by {
    gte_or_lt(a, b)
    if a >= b {
        a < b or a >= b
    } else {
        a < b
        a < b or a >= b
    }
}

/// Either `a < b` or `a >= b`.
theorem lt_or_gte[L: LinearOrder](a: L, b: L) {
    a < b or a >= b
} by {
    lt_or_ge(a, b)
}

/// Either `a > b` or `a <= b`.
theorem gt_or_lte[L: LinearOrder](a: L, b: L) {
    a > b or a <= b
} by {
    gt_or_le(a, b)
}

theorem max_imp_gte[L: LinearOrder](a: L, b: L) {
    a.max(b) >= a and a.max(b) >= b
} by {
    // Funny that the prover needs help here (says Prover status: Exhausted?)
    if a.max(b) = b {
        a.max(b) >= b
    } else {
        a.max(b) >= b
    }
}

theorem min_imp_lte[L: LinearOrder](a: L, b: L) {
    a.min(b) <= a and a.min(b) <= b
} by {
    if a.min(b) = a {
        a.min(b) <= b
    } else {
        a.min(b) <= b
    }
}

theorem lt_imp_min[L: LinearOrder](a: L, b: L) {
    a < b implies a.min(b) = a
}

theorem not_lt_imp_min[L: LinearOrder](a: L, b: L) {
    not a < b implies a.min(b) = b
}

theorem gt_imp_min[L: LinearOrder](a: L, b: L) {
    a > b implies a.min(b) = b
}

theorem not_gt_imp_min[L: LinearOrder](a: L, b: L) {
    not a > b implies a.min(b) = a
}

theorem lte_imp_min[L: LinearOrder](a: L, b: L) {
    a <= b implies a.min(b) = a
} by {
    if a <= b {
        lte_imp_not_gt(a, b)
        not a > b
        not_gt_imp_min(a, b)
        a.min(b) = a
    }
}

theorem gte_imp_min[L: LinearOrder](a: L, b: L) {
    a >= b implies a.min(b) = b
}

theorem not_gte_imp_min[L: LinearOrder](a: L, b: L) {
    not a >= b implies a.min(b) = a
}

theorem lt_imp_max[L: LinearOrder](a: L, b: L) {
    a < b implies a.max(b) = b
}

theorem not_lt_imp_max[L: LinearOrder](a: L, b: L) {
    not a < b implies a.max(b) = a
}

theorem gt_imp_max[L: LinearOrder](a: L, b: L) {
    a > b implies a.max(b) = a
}

theorem not_gt_imp_max[L: LinearOrder](a: L, b: L) {
    not a > b implies a.max(b) = b
}

theorem gte_imp_max[L: LinearOrder](a: L, b: L) {
    a >= b implies a.max(b) = a
}

theorem not_gte_imp_max[L: LinearOrder](a: L, b: L) {
    not a >= b implies a.max(b) = b
}

theorem min_is_one[L: LinearOrder](a: L, b: L) {
    a.min(b) = a or a.min(b) = b
}

theorem max_is_one[L: LinearOrder](a: L, b: L) {
    a.max(b) = a or a.max(b) = b
}

theorem min_symm[L: LinearOrder](a: L, b: L) {
    a.min(b) = b.min(a)
} by {
    if a <= b {
    } else {
        b.min(a) = a.min(b)
    }
}

theorem max_symm[L: LinearOrder](a: L, b: L) {
    a.max(b) = b.max(a)
} by {
    if a <= b {
    } else {
        b.max(a) = a.max(b)
    }
}

/// The minimum operation is commutative.
theorem min_comm[L: LinearOrder](a: L, b: L) {
    a.min(b) = b.min(a)
} by {
    min_symm(a, b)
}

/// The maximum operation is commutative.
theorem max_comm[L: LinearOrder](a: L, b: L) {
    a.max(b) = b.max(a)
} by {
    max_symm(a, b)
}

theorem min_lte_left[L: LinearOrder](a: L, b: L) {
    a.min(b) <= a
} by {
    min_imp_lte(a, b)
    a.min(b) <= a
}

/// A minimum is below its left argument.
theorem min_le_left[L: LinearOrder](a: L, b: L) {
    a.min(b) <= a
} by {
    min_lte_left(a, b)
}

theorem min_lte_right[L: LinearOrder](a: L, b: L) {
    a.min(b) <= b
} by {
    min_imp_lte(a, b)
    a.min(b) <= b
}

/// A minimum is below its right argument.
theorem min_le_right[L: LinearOrder](a: L, b: L) {
    a.min(b) <= b
} by {
    min_lte_right(a, b)
}

theorem lte_max_left[L: LinearOrder](a: L, b: L) {
    a <= a.max(b)
} by {
    max_imp_gte(a, b)
    a.max(b) >= a
    a <= a.max(b)
}

/// The left argument is below the maximum.
theorem le_max_left[L: LinearOrder](a: L, b: L) {
    a <= a.max(b)
} by {
    lte_max_left(a, b)
}

theorem lte_max_right[L: LinearOrder](a: L, b: L) {
    b <= a.max(b)
} by {
    max_imp_gte(a, b)
    a.max(b) >= b
    b <= a.max(b)
}

/// The right argument is below the maximum.
theorem le_max_right[L: LinearOrder](a: L, b: L) {
    b <= a.max(b)
} by {
    lte_max_right(a, b)
}

/// A minimum is below its left argument, in reverse-order form.
theorem min_gte_left[L: LinearOrder](a: L, b: L) {
    a >= a.min(b)
} by {
    min_lte_left(a, b)
    a >= a.min(b)
}

/// A minimum is below its right argument, in reverse-order form.
theorem min_gte_right[L: LinearOrder](a: L, b: L) {
    b >= a.min(b)
} by {
    min_lte_right(a, b)
    b >= a.min(b)
}

/// A maximum is above its left argument, in reverse-order form.
theorem max_gte_left[L: LinearOrder](a: L, b: L) {
    a.max(b) >= a
} by {
    lte_max_left(a, b)
    a.max(b) >= a
}

/// A maximum is above its right argument, in reverse-order form.
theorem max_gte_right[L: LinearOrder](a: L, b: L) {
    a.max(b) >= b
} by {
    lte_max_right(a, b)
    a.max(b) >= b
}

/// A minimum is below its left argument.
theorem min_le_of_left[L: LinearOrder](a: L, b: L) {
    a.min(b) <= a
} by {
    min_lte_left(a, b)
}

/// A minimum is below its right argument.
theorem min_le_of_right[L: LinearOrder](a: L, b: L) {
    a.min(b) <= b
} by {
    min_lte_right(a, b)
}

/// The left argument is below the maximum.
theorem le_max_of_left[L: LinearOrder](a: L, b: L) {
    a <= a.max(b)
} by {
    lte_max_left(a, b)
}

/// The right argument is below the maximum.
theorem le_max_of_right[L: LinearOrder](a: L, b: L) {
    b <= a.max(b)
} by {
    lte_max_right(a, b)
}

theorem min_idem[L: LinearOrder](a: L) {
    a.min(a) = a
} by {
    a <= a
    lte_imp_min(a, a)
}

theorem max_idem[L: LinearOrder](a: L) {
    a.max(a) = a
} by {
    a >= a
    gte_imp_max(a, a)
}

theorem min_eq_left_of_lte[L: LinearOrder](a: L, b: L) {
    a <= b implies a.min(b) = a
} by {
    if a <= b {
        lte_imp_min(a, b)
        a.min(b) = a
    }
}

theorem min_eq_right_of_gte[L: LinearOrder](a: L, b: L) {
    a >= b implies a.min(b) = b
} by {
    if a >= b {
        gte_imp_min(a, b)
        a.min(b) = b
    }
}

theorem max_eq_right_of_lte[L: LinearOrder](a: L, b: L) {
    a <= b implies a.max(b) = b
} by {
    if a <= b {
        eq_or_lt_of_lte(a, b)
        if a = b {
            b.max(a) = b
            max_symm(a, b)
            a.max(b) = b
        } else {
            a < b
            lt_imp_max(a, b)
            a.max(b) = b
        }
    }
}

theorem max_eq_left_of_gte[L: LinearOrder](a: L, b: L) {
    a >= b implies a.max(b) = a
} by {
    if a >= b {
        gte_imp_max(a, b)
        a.max(b) = a
    }
}

/// A strict comparison chooses the left argument as the minimum.
theorem min_eq_left_of_lt[L: LinearOrder](a: L, b: L) {
    a < b implies a.min(b) = a
} by {
    if a < b {
        a <= b
        min_eq_left_of_lte(a, b)
        a.min(b) = a
    }
}

/// A strict reverse comparison chooses the right argument as the minimum.
theorem min_eq_right_of_lt[L: LinearOrder](a: L, b: L) {
    b < a implies a.min(b) = b
} by {
    if b < a {
        a >= b
        min_eq_right_of_gte(a, b)
        a.min(b) = b
    }
}

/// A strict reverse comparison chooses the left argument as the maximum.
theorem max_eq_left_of_gt[L: LinearOrder](a: L, b: L) {
    a > b implies a.max(b) = a
} by {
    if a > b {
        a >= b
        max_eq_left_of_gte(a, b)
        a.max(b) = a
    }
}

/// A strict comparison chooses the right argument as the maximum.
theorem max_eq_right_of_gt[L: LinearOrder](a: L, b: L) {
    b > a implies a.max(b) = b
} by {
    if b > a {
        a <= b
        max_eq_right_of_lte(a, b)
        a.max(b) = b
    }
}

/// A strict comparison chooses the left argument as the minimum.
theorem min_of_lt[L: LinearOrder](a: L, b: L) {
    a < b implies a.min(b) = a
} by {
    if a < b {
        min_eq_left_of_lt(a, b)
        a.min(b) = a
    }
}

/// A strict reverse comparison chooses the right argument as the minimum.
theorem min_of_gt[L: LinearOrder](a: L, b: L) {
    a > b implies a.min(b) = b
} by {
    if a > b {
        min_eq_right_of_gte(a, b)
        a.min(b) = b
    }
}

/// A strict comparison chooses the right argument as the maximum.
theorem max_of_lt[L: LinearOrder](a: L, b: L) {
    a < b implies a.max(b) = b
} by {
    if a < b {
        max_eq_right_of_lte(a, b)
        a.max(b) = b
    }
}

/// A strict reverse comparison chooses the left argument as the maximum.
theorem max_of_gt[L: LinearOrder](a: L, b: L) {
    a > b implies a.max(b) = a
} by {
    if a > b {
        max_eq_left_of_gte(a, b)
        a.max(b) = a
    }
}

/// Failure of a strict reverse comparison chooses the left argument as the minimum.
theorem min_eq_left_of_not_gt[L: LinearOrder](a: L, b: L) {
    not (a > b) implies a.min(b) = a
} by {
    if not (a > b) {
        not_gt_imp_lte(a, b)
        min_eq_left_of_lte(a, b)
        a.min(b) = a
    }
}

/// Failure of a strict comparison chooses the right argument as the minimum.
theorem min_eq_right_of_not_lt[L: LinearOrder](a: L, b: L) {
    not (a < b) implies a.min(b) = b
} by {
    if not (a < b) {
        not_lt_imp_gte(a, b)
        min_eq_right_of_gte(a, b)
        a.min(b) = b
    }
}

/// Failure of a strict comparison chooses the left argument as the maximum.
theorem max_eq_left_of_not_lt[L: LinearOrder](a: L, b: L) {
    not (a < b) implies a.max(b) = a
} by {
    if not (a < b) {
        not_lt_imp_gte(a, b)
        max_eq_left_of_gte(a, b)
        a.max(b) = a
    }
}

/// Failure of a strict reverse comparison chooses the right argument as the maximum.
theorem max_eq_right_of_not_gt[L: LinearOrder](a: L, b: L) {
    not (a > b) implies a.max(b) = b
} by {
    if not (a > b) {
        not_gt_imp_lte(a, b)
        max_eq_right_of_lte(a, b)
        a.max(b) = b
    }
}

theorem min_absorb_max[L: LinearOrder](a: L, b: L) {
    a.min(a.max(b)) = a
} by {
    lte_max_left(a, b)
    lte_imp_min(a, a.max(b))
}

theorem max_absorb_min[L: LinearOrder](a: L, b: L) {
    a.max(a.min(b)) = a
} by {
    min_lte_left(a, b)
    gte_imp_max(a, a.min(b))
}

theorem min_eq_left_iff_lte[L: LinearOrder](a: L, b: L) {
    a.min(b) = a = (a <= b)
} by {
    if a.min(b) = a {
        min_imp_lte(a, b)
        a.min(b) <= b
        a <= b
    }
    if a <= b {
        min_eq_left_of_lte(a, b)
        a.min(b) = a
    }
    a.min(b) = a = (a <= b)
}

/// The minimum is the left argument exactly when the left argument is below the right.
theorem min_eq_left_iff_le[L: LinearOrder](a: L, b: L) {
    a.min(b) = a = (a <= b)
} by {
    min_eq_left_iff_lte(a, b)
}

theorem min_eq_right_iff_gte[L: LinearOrder](a: L, b: L) {
    a.min(b) = b = (a >= b)
} by {
    if a.min(b) = b {
        min_imp_lte(a, b)
        a.min(b) <= a
        a >= b
    }
    if a >= b {
        min_eq_right_of_gte(a, b)
        a.min(b) = b
    }
    a.min(b) = b = (a >= b)
}

/// The minimum is the right argument exactly when the left argument is above the right.
theorem min_eq_right_iff_ge[L: LinearOrder](a: L, b: L) {
    a.min(b) = b = (a >= b)
} by {
    min_eq_right_iff_gte(a, b)
}

theorem max_eq_right_iff_lte[L: LinearOrder](a: L, b: L) {
    a.max(b) = b = (a <= b)
} by {
    if a.max(b) = b {
        max_imp_gte(a, b)
        a.max(b) >= a
        a <= b
    }
    if a <= b {
        max_eq_right_of_lte(a, b)
        a.max(b) = b
    }
    a.max(b) = b = (a <= b)
}

/// The maximum is the right argument exactly when the left argument is below the right.
theorem max_eq_right_iff_le[L: LinearOrder](a: L, b: L) {
    a.max(b) = b = (a <= b)
} by {
    max_eq_right_iff_lte(a, b)
}

theorem max_eq_left_iff_gte[L: LinearOrder](a: L, b: L) {
    a.max(b) = a = (a >= b)
} by {
    if a.max(b) = a {
        max_imp_gte(a, b)
        a.max(b) >= b
        a >= b
    }
    if a >= b {
        max_eq_left_of_gte(a, b)
        a.max(b) = a
    }
    a.max(b) = a = (a >= b)
}

/// The maximum is the left argument exactly when the left argument is above the right.
theorem max_eq_left_iff_ge[L: LinearOrder](a: L, b: L) {
    a.max(b) = a = (a >= b)
} by {
    max_eq_left_iff_gte(a, b)
}

/// A value is below a minimum exactly when it is below both arguments.
theorem lt_min_imp[L: LinearOrder](c: L, a: L, b: L) {
    c < a.min(b) implies c < a and c < b
} by {
    if c < a.min(b) {
        min_lte_left(a, b)
        lt_of_lt_of_lte(c, a.min(b), a)
        c < a
        min_lte_right(a, b)
        lt_of_lt_of_lte(c, a.min(b), b)
        c < b
    }
}

/// Two strict upper bounds of a value give a strict upper bound by their minimum.
theorem lt_min_of_bounds[L: LinearOrder](c: L, a: L, b: L) {
    c < a and c < b implies c < a.min(b)
} by {
    if c < a and c < b {
        if a <= b {
            min_eq_left_of_lte(a, b)
            c < a.min(b)
        } else {
            a >= b
            min_eq_right_of_gte(a, b)
            c < a.min(b)
        }
    }
}

/// A value is below a minimum exactly when it is below both arguments.
theorem lt_min_iff[L: LinearOrder](c: L, a: L, b: L) {
    c < a.min(b) = (c < a and c < b)
} by {
    lt_min_imp(c, a, b)
    lt_min_of_bounds(c, a, b)
    c < a.min(b) = (c < a and c < b)
}

/// A strict lower bound of both arguments is a strict lower bound of their minimum.
theorem lt_both_imp_lt_min[L: LinearOrder](a: L, b: L, c: L) {
    a < b and a < c implies a < b.min(c)
} by {
    if a < b and a < c {
        lt_min_of_bounds(a, b, c)
        a < b.min(c)
    }
}

/// A strict lower bound of a minimum is a strict lower bound of the left argument.
theorem lt_min_imp_lt_left[L: LinearOrder](a: L, b: L, c: L) {
    a < b.min(c) implies a < b
} by {
    if a < b.min(c) {
        lt_min_imp(a, b, c)
        a < b
    }
}

/// A strict lower bound of a minimum is a strict lower bound of the right argument.
theorem lt_min_imp_lt_right[L: LinearOrder](a: L, b: L, c: L) {
    a < b.min(c) implies a < c
} by {
    if a < b.min(c) {
        lt_min_imp(a, b, c)
        a < c
    }
}

/// A maximum is below a value exactly when both arguments are below that value.
theorem max_lt_imp[L: LinearOrder](a: L, b: L, c: L) {
    a.max(b) < c implies a < c and b < c
} by {
    if a.max(b) < c {
        lte_max_left(a, b)
        lt_of_lte_of_lt(a, a.max(b), c)
        a < c
        lte_max_right(a, b)
        lt_of_lte_of_lt(b, a.max(b), c)
        b < c
    }
}

/// Two strict lower bounds of a value give a strict lower bound by their maximum.
theorem max_lt_of_upper_bounds[L: LinearOrder](a: L, b: L, c: L) {
    a < c and b < c implies a.max(b) < c
} by {
    if a < c and b < c {
        if a <= b {
            max_eq_right_of_lte(a, b)
            a.max(b) < c
        } else {
            a >= b
            max_eq_left_of_gte(a, b)
            a.max(b) < c
        }
    }
}

/// A maximum is below a value exactly when both arguments are below that value.
theorem max_lt_iff[L: LinearOrder](a: L, b: L, c: L) {
    a.max(b) < c = (a < c and b < c)
} by {
    max_lt_imp(a, b, c)
    max_lt_of_upper_bounds(a, b, c)
    a.max(b) < c = (a < c and b < c)
}

/// A strict upper bound of both arguments is a strict upper bound of their maximum.
theorem gt_both_imp_gt_max[L: LinearOrder](a: L, b: L, c: L) {
    a > b and a > c implies a > b.max(c)
} by {
    if a > b and a > c {
        b < a
        c < a
        max_lt_of_upper_bounds(b, c, a)
        b.max(c) < a
        a > b.max(c)
    }
}

/// A strict upper bound of a maximum is a strict upper bound of the left argument.
theorem gt_max_imp_gt_left[L: LinearOrder](a: L, b: L, c: L) {
    a > b.max(c) implies a > b
} by {
    if a > b.max(c) {
        b.max(c) < a
        max_lt_imp(b, c, a)
        b < a
        a > b
    }
}

/// A strict upper bound of a maximum is a strict upper bound of the right argument.
theorem gt_max_imp_gt_right[L: LinearOrder](a: L, b: L, c: L) {
    a > b.max(c) implies a > c
} by {
    if a > b.max(c) {
        b.max(c) < a
        max_lt_imp(b, c, a)
        c < a
        a > c
    }
}

/// A minimum is below a value exactly when at least one argument is below that value.
theorem min_lt_iff[L: LinearOrder](a: L, b: L, c: L) {
    a.min(b) < c = (a < c or b < c)
} by {
    if a.min(b) < c {
        min_is_one(a, b)
        if a.min(b) = a {
            a < c
            a < c or b < c
        } else {
            a.min(b) = b
            b < c
            a < c or b < c
        }
    }
    if a < c or b < c {
        if a < c {
            min_lte_left(a, b)
            lt_of_lte_of_lt(a.min(b), a, c)
            a.min(b) < c
        } else {
            min_lte_right(a, b)
            lt_of_lte_of_lt(a.min(b), b, c)
            a.min(b) < c
        }
    }
    a.min(b) < c = (a < c or b < c)
}

/// If the left argument is below a value, then the minimum is below that value.
theorem min_lt_of_left[L: LinearOrder](a: L, b: L, c: L) {
    a < c implies a.min(b) < c
} by {
    if a < c {
        min_lte_left(a, b)
        lt_of_lte_of_lt(a.min(b), a, c)
        a.min(b) < c
    }
}

/// If the right argument is below a value, then the minimum is below that value.
theorem min_lt_of_right[L: LinearOrder](a: L, b: L, c: L) {
    b < c implies a.min(b) < c
} by {
    if b < c {
        min_lte_right(a, b)
        lt_of_lte_of_lt(a.min(b), b, c)
        a.min(b) < c
    }
}

/// If a minimum is below a value, then one argument is below that value.
theorem min_lt_imp_left_or_right[L: LinearOrder](a: L, b: L, c: L) {
    a.min(b) < c implies a < c or b < c
} by {
    if a.min(b) < c {
        min_lt_iff(a, b, c)
        a < c or b < c
    }
}

/// If one argument is below a value, then the minimum is below that value.
theorem min_lt_of_left_or_right[L: LinearOrder](a: L, b: L, c: L) {
    a < c or b < c implies a.min(b) < c
} by {
    if a < c or b < c {
        min_lt_iff(a, b, c)
        a.min(b) < c
    }
}

/// A value is below a maximum exactly when it is below at least one argument.
theorem lt_max_iff[L: LinearOrder](c: L, a: L, b: L) {
    c < a.max(b) = (c < a or c < b)
} by {
    if c < a.max(b) {
        max_is_one(a, b)
        if a.max(b) = a {
            c < a
            c < a or c < b
        } else {
            a.max(b) = b
            c < b
            c < a or c < b
        }
    }
    if c < a or c < b {
        if c < a {
            lte_max_left(a, b)
            lt_of_lt_of_lte(c, a, a.max(b))
            c < a.max(b)
        } else {
            lte_max_right(a, b)
            lt_of_lt_of_lte(c, b, a.max(b))
            c < a.max(b)
        }
    }
    c < a.max(b) = (c < a or c < b)
}

/// If a value is below the left argument, then it is below the maximum.
theorem lt_max_of_left[L: LinearOrder](c: L, a: L, b: L) {
    c < a implies c < a.max(b)
} by {
    if c < a {
        lte_max_left(a, b)
        lt_of_lt_of_lte(c, a, a.max(b))
        c < a.max(b)
    }
}

/// If a value is below the right argument, then it is below the maximum.
theorem lt_max_of_right[L: LinearOrder](c: L, a: L, b: L) {
    c < b implies c < a.max(b)
} by {
    if c < b {
        lte_max_right(a, b)
        lt_of_lt_of_lte(c, b, a.max(b))
        c < a.max(b)
    }
}

/// If a value is below a maximum, then it is below one argument.
theorem lt_max_imp_left_or_right[L: LinearOrder](c: L, a: L, b: L) {
    c < a.max(b) implies c < a or c < b
} by {
    if c < a.max(b) {
        lt_max_iff(c, a, b)
        c < a or c < b
    }
}

/// If a value is below one argument, then it is below the maximum.
theorem lt_max_of_left_or_right[L: LinearOrder](c: L, a: L, b: L) {
    c < a or c < b implies c < a.max(b)
} by {
    if c < a or c < b {
        lt_max_iff(c, a, b)
        c < a.max(b)
    }
}

/// A value is below a minimum only if it is below both arguments.
theorem lte_min_imp[L: LinearOrder](c: L, a: L, b: L) {
    c <= a.min(b) implies c <= a and c <= b
} by {
    if c <= a.min(b) {
        min_lte_left(a, b)
        lte_trans(c, a.min(b), a)
        c <= a
        min_lte_right(a, b)
        lte_trans(c, a.min(b), b)
        c <= b
    }
}

/// Two upper bounds of a value give an upper bound by their minimum.
theorem lte_min_of_bounds[L: LinearOrder](c: L, a: L, b: L) {
    c <= a and c <= b implies c <= a.min(b)
} by {
    if c <= a and c <= b {
        if a <= b {
            min_eq_left_of_lte(a, b)
            c <= a.min(b)
        } else {
            a >= b
            min_eq_right_of_gte(a, b)
            c <= a.min(b)
        }
    }
}

/// A value is below a minimum exactly when it is below both arguments.
theorem lte_min_iff[L: LinearOrder](c: L, a: L, b: L) {
    c <= a.min(b) = (c <= a and c <= b)
} by {
    lte_min_imp(c, a, b)
    lte_min_of_bounds(c, a, b)
    c <= a.min(b) = (c <= a and c <= b)
}

/// A value is below a minimum exactly when it is below both arguments.
theorem le_min_iff[L: LinearOrder](c: L, a: L, b: L) {
    c <= a.min(b) = (c <= a and c <= b)
} by {
    lte_min_iff(c, a, b)
}

/// A lower bound of both arguments is a lower bound of their minimum.
theorem le_min_of_le_left_of_le_right[L: LinearOrder](c: L, a: L, b: L) {
    c <= a and c <= b implies c <= a.min(b)
} by {
    if c <= a and c <= b {
        lte_min_of_bounds(c, a, b)
        c <= a.min(b)
    }
}

/// A lower bound of a minimum is a lower bound of the left argument.
theorem le_left_of_le_min[L: LinearOrder](c: L, a: L, b: L) {
    c <= a.min(b) implies c <= a
} by {
    if c <= a.min(b) {
        lte_min_imp(c, a, b)
        c <= a
    }
}

/// A lower bound of a minimum is a lower bound of the right argument.
theorem le_right_of_le_min[L: LinearOrder](c: L, a: L, b: L) {
    c <= a.min(b) implies c <= b
} by {
    if c <= a.min(b) {
        lte_min_imp(c, a, b)
        c <= b
    }
}

/// If the left argument is below a value, then the minimum is below that value.
theorem min_lte_of_left[L: LinearOrder](a: L, b: L, c: L) {
    a <= c implies a.min(b) <= c
} by {
    if a <= c {
        min_lte_left(a, b)
        lte_trans(a.min(b), a, c)
        a.min(b) <= c
    }
}

/// If the right argument is below a value, then the minimum is below that value.
theorem min_lte_of_right[L: LinearOrder](a: L, b: L, c: L) {
    b <= c implies a.min(b) <= c
} by {
    if b <= c {
        min_lte_right(a, b)
        lte_trans(a.min(b), b, c)
        a.min(b) <= c
    }
}

/// If a minimum is below a value, then one argument is below that value.
theorem min_lte_imp_left_or_right[L: LinearOrder](a: L, b: L, c: L) {
    a.min(b) <= c implies a <= c or b <= c
} by {
    if a.min(b) <= c {
        min_is_one(a, b)
        if a.min(b) = a {
            a <= c
            a <= c or b <= c
        } else {
            a.min(b) = b
            b <= c
            a <= c or b <= c
        }
    }
}

/// A minimum is above a value only if both arguments are above that value.
theorem min_gte_imp[L: LinearOrder](a: L, b: L, c: L) {
    a.min(b) >= c implies a >= c and b >= c
} by {
    if a.min(b) >= c {
        lte_min_imp(c, a, b)
        c <= a and c <= b
        a >= c
        b >= c
        a >= c and b >= c
    }
}

/// Two lower bounds of arguments give a lower bound of their minimum.
theorem min_gte_of_bounds[L: LinearOrder](a: L, b: L, c: L) {
    a >= c and b >= c implies a.min(b) >= c
} by {
    if a >= c and b >= c {
        lte_min_of_bounds(c, a, b)
        c <= a.min(b)
        a.min(b) >= c
    }
}

/// A minimum is above a value exactly when both arguments are above that value.
theorem min_gte_iff[L: LinearOrder](a: L, b: L, c: L) {
    a.min(b) >= c = (a >= c and b >= c)
} by {
    min_gte_imp(a, b, c)
    min_gte_of_bounds(a, b, c)
    a.min(b) >= c = (a >= c and b >= c)
}

/// A minimum is above a value exactly when both arguments are above that value.
theorem min_ge_iff[L: LinearOrder](a: L, b: L, c: L) {
    a.min(b) >= c = (a >= c and b >= c)
} by {
    min_gte_iff(a, b, c)
}

/// Two upper bounds of a value give an upper bound by their minimum.
theorem min_ge_of_ge_left_of_ge_right[L: LinearOrder](a: L, b: L, c: L) {
    a >= c and b >= c implies a.min(b) >= c
} by {
    if a >= c and b >= c {
        min_gte_of_bounds(a, b, c)
        a.min(b) >= c
    }
}

/// If a minimum is above a value, then the left argument is above that value.
theorem ge_left_of_min_ge[L: LinearOrder](a: L, b: L, c: L) {
    a.min(b) >= c implies a >= c
} by {
    if a.min(b) >= c {
        min_gte_imp(a, b, c)
        a >= c
    }
}

/// If a minimum is above a value, then the right argument is above that value.
theorem ge_right_of_min_ge[L: LinearOrder](a: L, b: L, c: L) {
    a.min(b) >= c implies b >= c
} by {
    if a.min(b) >= c {
        min_gte_imp(a, b, c)
        b >= c
    }
}

/// A maximum is below a value only if both arguments are below that value.
theorem max_lte_imp[L: LinearOrder](a: L, b: L, c: L) {
    a.max(b) <= c implies a <= c and b <= c
} by {
    if a.max(b) <= c {
        max_imp_gte(a, b)
        a.max(b) >= a
        a <= a.max(b)
        lte_trans(a, a.max(b), c)
        a <= c
        a.max(b) >= b
        b <= a.max(b)
        lte_trans(b, a.max(b), c)
        b <= c
    }
}

/// Two upper bounds of arguments give an upper bound of their maximum.
theorem max_lte_of_upper_bounds[L: LinearOrder](a: L, b: L, c: L) {
    a <= c and b <= c implies a.max(b) <= c
} by {
    if a <= c and b <= c {
        if a <= b {
            max_eq_right_of_lte(a, b)
            a.max(b) <= c
        } else {
            a >= b
            max_eq_left_of_gte(a, b)
            a.max(b) <= c
        }
    }
}

/// A maximum is below a value exactly when both arguments are below that value.
theorem max_lte_iff[L: LinearOrder](a: L, b: L, c: L) {
    a.max(b) <= c = (a <= c and b <= c)
} by {
    max_lte_imp(a, b, c)
    max_lte_of_upper_bounds(a, b, c)
    a.max(b) <= c = (a <= c and b <= c)
}

/// A maximum is below a value exactly when both arguments are below that value.
theorem max_le_iff[L: LinearOrder](a: L, b: L, c: L) {
    a.max(b) <= c = (a <= c and b <= c)
} by {
    max_lte_iff(a, b, c)
}

/// Two upper bounds give an upper bound of the maximum.
theorem max_le_of_le_left_of_le_right[L: LinearOrder](a: L, b: L, c: L) {
    a <= c and b <= c implies a.max(b) <= c
} by {
    if a <= c and b <= c {
        max_lte_of_upper_bounds(a, b, c)
        a.max(b) <= c
    }
}

/// If a maximum is below a value, then the left argument is below that value.
theorem le_left_of_max_le[L: LinearOrder](a: L, b: L, c: L) {
    a.max(b) <= c implies a <= c
} by {
    if a.max(b) <= c {
        max_lte_imp(a, b, c)
        a <= c
    }
}

/// If a maximum is below a value, then the right argument is below that value.
theorem le_right_of_max_le[L: LinearOrder](a: L, b: L, c: L) {
    a.max(b) <= c implies b <= c
} by {
    if a.max(b) <= c {
        max_lte_imp(a, b, c)
        b <= c
    }
}

/// If a value is below the left argument, then it is below the maximum.
theorem lte_max_of_left[L: LinearOrder](c: L, a: L, b: L) {
    c <= a implies c <= a.max(b)
} by {
    if c <= a {
        lte_max_left(a, b)
        lte_trans(c, a, a.max(b))
        c <= a.max(b)
    }
}

/// If a value is below the right argument, then it is below the maximum.
theorem lte_max_of_right[L: LinearOrder](c: L, a: L, b: L) {
    c <= b implies c <= a.max(b)
} by {
    if c <= b {
        lte_max_right(a, b)
        lte_trans(c, b, a.max(b))
        c <= a.max(b)
    }
}

/// If a value is below a maximum, then it is below one argument.
theorem lte_max_imp_left_or_right[L: LinearOrder](c: L, a: L, b: L) {
    c <= a.max(b) implies c <= a or c <= b
} by {
    if c <= a.max(b) {
        max_is_one(a, b)
        if a.max(b) = a {
            c <= a
            c <= a or c <= b
        } else {
            a.max(b) = b
            c <= b
            c <= a or c <= b
        }
    }
}

/// The minimum operation is monotone in both arguments.
theorem min_lte_min[L: LinearOrder](a: L, b: L, c: L, d: L) {
    a <= b and c <= d implies a.min(c) <= b.min(d)
} by {
    if a <= b and c <= d {
        min_lte_left(a, c)
        a.min(c) <= a
        lte_trans(a.min(c), a, b)
        a.min(c) <= b
        min_lte_right(a, c)
        a.min(c) <= c
        lte_trans(a.min(c), c, d)
        a.min(c) <= d
        lte_min_of_bounds(a.min(c), b, d)
        a.min(c) <= b.min(d)
    }
}

/// The minimum operation is monotone in both arguments.
theorem min_le_min[L: LinearOrder](a: L, b: L, c: L, d: L) {
    a <= b and c <= d implies a.min(c) <= b.min(d)
} by {
    if a <= b and c <= d {
        min_lte_min(a, b, c, d)
        a.min(c) <= b.min(d)
    }
}

/// The maximum operation is monotone in both arguments.
theorem max_lte_max[L: LinearOrder](a: L, b: L, c: L, d: L) {
    a <= b and c <= d implies a.max(c) <= b.max(d)
} by {
    if a <= b and c <= d {
        lte_max_left(b, d)
        b <= b.max(d)
        lte_trans(a, b, b.max(d))
        a <= b.max(d)
        lte_max_right(b, d)
        d <= b.max(d)
        lte_trans(c, d, b.max(d))
        c <= b.max(d)
        max_lte_of_upper_bounds(a, c, b.max(d))
        a.max(c) <= b.max(d)
    }
}

/// The maximum operation is monotone in both arguments.
theorem max_le_max[L: LinearOrder](a: L, b: L, c: L, d: L) {
    a <= b and c <= d implies a.max(c) <= b.max(d)
} by {
    if a <= b and c <= d {
        max_lte_max(a, b, c, d)
        a.max(c) <= b.max(d)
    }
}

/// The minimum operation is monotone in the left argument.
theorem min_lte_min_left[L: LinearOrder](a: L, b: L, c: L) {
    a <= b implies a.min(c) <= b.min(c)
} by {
    if a <= b {
        min_lte_min(a, b, c, c)
        a.min(c) <= b.min(c)
    }
}

/// The minimum operation is monotone in the left argument.
theorem min_le_min_left[L: LinearOrder](a: L, b: L, c: L) {
    a <= b implies a.min(c) <= b.min(c)
} by {
    if a <= b {
        min_lte_min_left(a, b, c)
        a.min(c) <= b.min(c)
    }
}

/// The minimum operation is monotone in the right argument.
theorem min_lte_min_right[L: LinearOrder](a: L, b: L, c: L) {
    b <= c implies a.min(b) <= a.min(c)
} by {
    if b <= c {
        min_lte_min(a, a, b, c)
        a.min(b) <= a.min(c)
    }
}

/// The minimum operation is monotone in the right argument.
theorem min_le_min_right[L: LinearOrder](a: L, b: L, c: L) {
    b <= c implies a.min(b) <= a.min(c)
} by {
    if b <= c {
        min_lte_min_right(a, b, c)
        a.min(b) <= a.min(c)
    }
}

/// The maximum operation is monotone in the left argument.
theorem max_lte_max_left[L: LinearOrder](a: L, b: L, c: L) {
    a <= b implies a.max(c) <= b.max(c)
} by {
    if a <= b {
        max_lte_max(a, b, c, c)
        a.max(c) <= b.max(c)
    }
}

/// The maximum operation is monotone in the left argument.
theorem max_le_max_left[L: LinearOrder](a: L, b: L, c: L) {
    a <= b implies a.max(c) <= b.max(c)
} by {
    if a <= b {
        max_lte_max_left(a, b, c)
        a.max(c) <= b.max(c)
    }
}

/// The maximum operation is monotone in the right argument.
theorem max_lte_max_right[L: LinearOrder](a: L, b: L, c: L) {
    b <= c implies a.max(b) <= a.max(c)
} by {
    if b <= c {
        max_lte_max(a, a, b, c)
        a.max(b) <= a.max(c)
    }
}

/// The maximum operation is monotone in the right argument.
theorem max_le_max_right[L: LinearOrder](a: L, b: L, c: L) {
    b <= c implies a.max(b) <= a.max(c)
} by {
    if b <= c {
        max_lte_max_right(a, b, c)
        a.max(b) <= a.max(c)
    }
}

/// The minimum operation is monotone for reverse-order bounds.
theorem min_gte_min[L: LinearOrder](a: L, b: L, c: L, d: L) {
    a >= b and c >= d implies a.min(c) >= b.min(d)
} by {
    if a >= b and c >= d {
        b <= a
        d <= c
        min_lte_min(b, a, d, c)
        b.min(d) <= a.min(c)
        a.min(c) >= b.min(d)
    }
}

/// The minimum operation is monotone for reverse-order bounds.
theorem min_ge_min[L: LinearOrder](a: L, b: L, c: L, d: L) {
    a >= b and c >= d implies a.min(c) >= b.min(d)
} by {
    if a >= b and c >= d {
        min_gte_min(a, b, c, d)
        a.min(c) >= b.min(d)
    }
}

/// The maximum operation is monotone for reverse-order bounds.
theorem max_gte_max[L: LinearOrder](a: L, b: L, c: L, d: L) {
    a >= b and c >= d implies a.max(c) >= b.max(d)
} by {
    if a >= b and c >= d {
        b <= a
        d <= c
        max_lte_max(b, a, d, c)
        b.max(d) <= a.max(c)
        a.max(c) >= b.max(d)
    }
}

/// The maximum operation is monotone for reverse-order bounds.
theorem max_ge_max[L: LinearOrder](a: L, b: L, c: L, d: L) {
    a >= b and c >= d implies a.max(c) >= b.max(d)
} by {
    if a >= b and c >= d {
        max_gte_max(a, b, c, d)
        a.max(c) >= b.max(d)
    }
}

/// The minimum operation is monotone in the left argument for reverse-order bounds.
theorem min_ge_min_left[L: LinearOrder](a: L, b: L, c: L) {
    a >= b implies a.min(c) >= b.min(c)
} by {
    if a >= b {
        min_gte_min(a, b, c, c)
        a.min(c) >= b.min(c)
    }
}

/// The minimum operation is monotone in the right argument for reverse-order bounds.
theorem min_ge_min_right[L: LinearOrder](a: L, b: L, c: L) {
    b >= c implies a.min(b) >= a.min(c)
} by {
    if b >= c {
        min_gte_min(a, a, b, c)
        a.min(b) >= a.min(c)
    }
}

/// The maximum operation is monotone in the left argument for reverse-order bounds.
theorem max_ge_max_left[L: LinearOrder](a: L, b: L, c: L) {
    a >= b implies a.max(c) >= b.max(c)
} by {
    if a >= b {
        max_gte_max(a, b, c, c)
        a.max(c) >= b.max(c)
    }
}

/// The maximum operation is monotone in the right argument for reverse-order bounds.
theorem max_ge_max_right[L: LinearOrder](a: L, b: L, c: L) {
    b >= c implies a.max(b) >= a.max(c)
} by {
    if b >= c {
        max_gte_max(a, a, b, c)
        a.max(b) >= a.max(c)
    }
}

/// A minimum is below a value only if at least one argument is below that value.
theorem min_lte_imp[L: LinearOrder](a: L, b: L, c: L) {
    a.min(b) <= c implies a <= c or b <= c
} by {
    if a.min(b) <= c {
        min_is_one(a, b)
        if a.min(b) = a {
            a <= c
            a <= c or b <= c
        } else {
            a.min(b) = b
            b <= c
            a <= c or b <= c
        }
    }
}

/// A lower argument gives a lower minimum.
theorem min_lte_of_left_or_right[L: LinearOrder](a: L, b: L, c: L) {
    a <= c or b <= c implies a.min(b) <= c
} by {
    if a <= c or b <= c {
        if a <= c {
            min_lte_left(a, b)
            a.min(b) <= a
            lte_trans(a.min(b), a, c)
            a.min(b) <= c
        } else {
            min_lte_right(a, b)
            a.min(b) <= b
            lte_trans(a.min(b), b, c)
            a.min(b) <= c
        }
    }
}

/// A minimum is below a value exactly when at least one argument is below that value.
theorem min_lte_iff[L: LinearOrder](a: L, b: L, c: L) {
    a.min(b) <= c = (a <= c or b <= c)
} by {
    min_lte_imp(a, b, c)
    min_lte_of_left_or_right(a, b, c)
    a.min(b) <= c = (a <= c or b <= c)
}

/// A value is below a maximum only if it is below at least one argument.
theorem lte_max_imp[L: LinearOrder](c: L, a: L, b: L) {
    c <= a.max(b) implies c <= a or c <= b
} by {
    if c <= a.max(b) {
        max_is_one(a, b)
        if a.max(b) = a {
            c <= a
            c <= a or c <= b
        } else {
            a.max(b) = b
            c <= b
            c <= a or c <= b
        }
    }
}

/// A lower bound of one argument gives a lower bound of the maximum.
theorem lte_max_of_left_or_right[L: LinearOrder](c: L, a: L, b: L) {
    c <= a or c <= b implies c <= a.max(b)
} by {
    if c <= a or c <= b {
        if c <= a {
            lte_max_left(a, b)
            lte_trans(c, a, a.max(b))
            c <= a.max(b)
        } else {
            lte_max_right(a, b)
            lte_trans(c, b, a.max(b))
            c <= a.max(b)
        }
    }
}

/// A value is below a maximum exactly when it is below at least one argument.
theorem lte_max_iff[L: LinearOrder](c: L, a: L, b: L) {
    c <= a.max(b) = (c <= a or c <= b)
} by {
    lte_max_imp(c, a, b)
    lte_max_of_left_or_right(c, a, b)
    c <= a.max(b) = (c <= a or c <= b)
}

/// A value is below a maximum exactly when it is below at least one argument.
theorem le_max_iff[L: LinearOrder](c: L, a: L, b: L) {
    c <= a.max(b) = (c <= a or c <= b)
} by {
    lte_max_iff(c, a, b)
}

/// A lower bound of one argument gives a lower bound of the maximum.
theorem le_max_of_le_left[L: LinearOrder](c: L, a: L, b: L) {
    c <= a implies c <= a.max(b)
} by {
    if c <= a {
        lte_max_left(a, b)
        lte_trans(c, a, a.max(b))
        c <= a.max(b)
    }
}

/// A lower bound of one argument gives a lower bound of the maximum.
theorem le_max_of_le_right[L: LinearOrder](c: L, a: L, b: L) {
    c <= b implies c <= a.max(b)
} by {
    if c <= b {
        lte_max_right(a, b)
        lte_trans(c, b, a.max(b))
        c <= a.max(b)
    }
}

/// A maximum is above a value only if at least one argument is above that value.
theorem max_gte_imp[L: LinearOrder](a: L, b: L, c: L) {
    a.max(b) >= c implies a >= c or b >= c
} by {
    if a.max(b) >= c {
        lte_max_imp(c, a, b)
        c <= a or c <= b
        if c <= a {
            a >= c
            a >= c or b >= c
        } else {
            c <= b
            b >= c
            a >= c or b >= c
        }
    }
}

/// A lower bound of one argument gives a lower bound of the maximum.
theorem max_gte_of_left_or_right[L: LinearOrder](a: L, b: L, c: L) {
    a >= c or b >= c implies a.max(b) >= c
} by {
    if a >= c or b >= c {
        if a >= c {
            lte_max_of_left_or_right(c, a, b)
            a.max(b) >= c
        } else {
            b >= c
            lte_max_of_left_or_right(c, a, b)
            a.max(b) >= c
        }
    }
}

/// A maximum is above a value exactly when at least one argument is above that value.
theorem max_gte_iff[L: LinearOrder](a: L, b: L, c: L) {
    a.max(b) >= c = (a >= c or b >= c)
} by {
    max_gte_imp(a, b, c)
    max_gte_of_left_or_right(a, b, c)
    a.max(b) >= c = (a >= c or b >= c)
}

/// A maximum is above a value exactly when one argument is above that value.
theorem max_ge_iff[L: LinearOrder](a: L, b: L, c: L) {
    a.max(b) >= c = (a >= c or b >= c)
} by {
    max_gte_iff(a, b, c)
}

/// A lower bound of the left argument is a lower bound of the maximum.
theorem max_ge_of_ge_left[L: LinearOrder](a: L, b: L, c: L) {
    a >= c implies a.max(b) >= c
} by {
    if a >= c {
        le_max_of_le_left(c, a, b)
        a.max(b) >= c
    }
}

/// A lower bound of the right argument is a lower bound of the maximum.
theorem max_ge_of_ge_right[L: LinearOrder](a: L, b: L, c: L) {
    b >= c implies a.max(b) >= c
} by {
    if b >= c {
        le_max_of_le_right(c, a, b)
        a.max(b) >= c
    }
}

/// Two elements have a common upper bound.
theorem exists_common_upper_bound[L: LinearOrder](a: L, b: L) {
    exists(n: L) {
        a <= n and b <= n
    }
} by {
    let n = a.max(b)
    lte_max_left(a, b)
    a <= n
    lte_max_right(a, b)
    b <= n
}

/// Two elements have a common lower bound.
theorem exists_common_lower_bound[L: LinearOrder](a: L, b: L) {
    exists(n: L) {
        n <= a and n <= b
    }
} by {
    let n = a.min(b)
    min_lte_left(a, b)
    n <= a
    min_lte_right(a, b)
    n <= b
}

/// Two elements have a common upper bound, written with reverse comparisons.
theorem exists_common_ge_bound[L: LinearOrder](a: L, b: L) {
    exists(n: L) {
        n >= a and n >= b
    }
} by {
    let n = a.max(b)
    lte_max_left(a, b)
    n >= a
    lte_max_right(a, b)
    n >= b
}

/// Two elements have a common lower bound, written with reverse comparisons.
theorem exists_common_le_bound[L: LinearOrder](a: L, b: L) {
    exists(n: L) {
        a >= n and b >= n
    }
} by {
    let n = a.min(b)
    min_lte_left(a, b)
    a >= n
    min_lte_right(a, b)
    b >= n
}

/// Three elements have a common upper bound.
theorem exists_common_upper_bound_three[L: LinearOrder](a: L, b: L, c: L) {
    exists(n: L) {
        a <= n and b <= n and c <= n
    }
} by {
    let m = a.max(b)
    let n = m.max(c)
    lte_max_left(a, b)
    a <= m
    lte_max_left(m, c)
    m <= n
    lte_trans(a, m, n)
    a <= n
    lte_max_right(a, b)
    b <= m
    lte_trans(b, m, n)
    b <= n
    lte_max_right(m, c)
    c <= n
}

/// Three elements have a common lower bound.
theorem exists_common_lower_bound_three[L: LinearOrder](a: L, b: L, c: L) {
    exists(n: L) {
        n <= a and n <= b and n <= c
    }
} by {
    let m = a.min(b)
    let n = m.min(c)
    min_lte_left(m, c)
    n <= m
    min_lte_left(a, b)
    m <= a
    lte_trans(n, m, a)
    n <= a
    min_lte_right(a, b)
    m <= b
    lte_trans(n, m, b)
    n <= b
    min_lte_right(m, c)
    n <= c
}

/// Three elements have a common upper bound, written with reverse comparisons.
theorem exists_common_ge_bound_three[L: LinearOrder](a: L, b: L, c: L) {
    exists(n: L) {
        n >= a and n >= b and n >= c
    }
} by {
    let m = a.max(b)
    let n = m.max(c)
    lte_max_left(a, b)
    a <= m
    lte_max_left(m, c)
    m <= n
    lte_trans(a, m, n)
    n >= a
    lte_max_right(a, b)
    b <= m
    lte_trans(b, m, n)
    n >= b
    lte_max_right(m, c)
    n >= c
}

/// Three elements have a common lower bound, written with reverse comparisons.
theorem exists_common_le_bound_three[L: LinearOrder](a: L, b: L, c: L) {
    exists(n: L) {
        a >= n and b >= n and c >= n
    }
} by {
    let m = a.min(b)
    let n = m.min(c)
    min_lte_left(m, c)
    n <= m
    min_lte_left(a, b)
    m <= a
    lte_trans(n, m, a)
    a >= n
    min_lte_right(a, b)
    m <= b
    lte_trans(n, m, b)
    b >= n
    min_lte_right(m, c)
    c >= n
}

/// The minimum operation is associative.
theorem min_assoc[L: LinearOrder](a: L, b: L, c: L) {
    a.min(b.min(c)) = a.min(b).min(c)
} by {
    let left = a.min(b.min(c))
    let right = a.min(b).min(c)
    min_lte_left(a, b.min(c))
    left <= a
    min_lte_right(a, b.min(c))
    left <= b.min(c)
    min_lte_left(b, c)
    b.min(c) <= b
    lte_trans(left, b.min(c), b)
    left <= b
    lte_min_of_bounds(left, a, b)
    left <= a.min(b)
    min_lte_right(b, c)
    b.min(c) <= c
    lte_trans(left, b.min(c), c)
    left <= c
    lte_min_of_bounds(left, a.min(b), c)
    left <= right
    min_lte_left(a.min(b), c)
    right <= a.min(b)
    min_lte_left(a, b)
    a.min(b) <= a
    lte_trans(right, a.min(b), a)
    right <= a
    min_lte_right(a, b)
    a.min(b) <= b
    lte_trans(right, a.min(b), b)
    right <= b
    min_lte_right(a.min(b), c)
    right <= c
    lte_min_of_bounds(right, b, c)
    right <= b.min(c)
    lte_min_of_bounds(right, a, b.min(c))
    right <= left
    lte_antisymm(left, right)
}

/// The maximum operation is associative.
theorem max_assoc[L: LinearOrder](a: L, b: L, c: L) {
    a.max(b.max(c)) = a.max(b).max(c)
} by {
    let left = a.max(b.max(c))
    let right = a.max(b).max(c)
    lte_max_left(a, b.max(c))
    a <= left
    lte_max_right(a, b.max(c))
    b.max(c) <= left
    lte_max_left(b, c)
    b <= b.max(c)
    lte_trans(b, b.max(c), left)
    b <= left
    max_lte_of_upper_bounds(a, b, left)
    a.max(b) <= left
    lte_max_right(b, c)
    c <= b.max(c)
    lte_trans(c, b.max(c), left)
    c <= left
    max_lte_of_upper_bounds(a.max(b), c, left)
    right <= left
    lte_max_left(a.max(b), c)
    a.max(b) <= right
    lte_max_left(a, b)
    a <= a.max(b)
    lte_trans(a, a.max(b), right)
    a <= right
    lte_max_right(a, b)
    b <= a.max(b)
    lte_trans(b, a.max(b), right)
    b <= right
    lte_max_right(a.max(b), c)
    c <= right
    max_lte_of_upper_bounds(b, c, right)
    b.max(c) <= right
    max_lte_of_upper_bounds(a, b.max(c), right)
    left <= right
    lte_antisymm(left, right)
}

/// The minimum operation is associative in the reversed nesting.
theorem min_assoc_rev[L: LinearOrder](a: L, b: L, c: L) {
    a.min(b).min(c) = a.min(b.min(c))
} by {
    min_assoc(a, b, c)
}

/// The maximum operation is associative in the reversed nesting.
theorem max_assoc_rev[L: LinearOrder](a: L, b: L, c: L) {
    a.max(b).max(c) = a.max(b.max(c))
} by {
    max_assoc(a, b, c)
}

/// The outer left argument of a nested minimum may be exchanged.
theorem min_left_comm[L: LinearOrder](a: L, b: L, c: L) {
    a.min(b.min(c)) = b.min(a.min(c))
} by {
    min_assoc(a, b, c)
    min_comm(a, b)
    min_assoc(b, a, c)
}

/// The outer left argument of a nested maximum may be exchanged.
theorem max_left_comm[L: LinearOrder](a: L, b: L, c: L) {
    a.max(b.max(c)) = b.max(a.max(c))
} by {
    max_assoc(a, b, c)
    max_comm(a, b)
    max_assoc(b, a, c)
}

/// The two right arguments of an iterated minimum may be exchanged.
theorem min_right_comm[L: LinearOrder](a: L, b: L, c: L) {
    a.min(b).min(c) = a.min(c).min(b)
} by {
    min_assoc_rev(a, b, c)
    min_comm(b, c)
    min_assoc(a, c, b)
}

/// The two right arguments of an iterated maximum may be exchanged.
theorem max_right_comm[L: LinearOrder](a: L, b: L, c: L) {
    a.max(b).max(c) = a.max(c).max(b)
} by {
    max_assoc_rev(a, b, c)
    max_comm(b, c)
    max_assoc(a, c, b)
}

/// A minimum absorbs a maximum with the same right argument.
theorem min_absorb_max_right[L: LinearOrder](a: L, b: L) {
    a.min(b.max(a)) = a
} by {
    max_comm(b, a)
    min_absorb_max(a, b)
}

/// A maximum absorbs a minimum with the same right argument.
theorem max_absorb_min_right[L: LinearOrder](a: L, b: L) {
    a.max(b.min(a)) = a
} by {
    min_comm(b, a)
    max_absorb_min(a, b)
}

/// Minimum distributes over maximum on the left.
theorem min_max_distrib_left[L: LinearOrder](a: L, b: L, c: L) {
    a.min(b.max(c)) = a.min(b).max(a.min(c))
} by {
    if b <= c {
        max_eq_right_of_lte(b, c)
        b.max(c) = c
        min_lte_min_right(a, b, c)
        a.min(b) <= a.min(c)
        max_eq_right_of_lte(a.min(b), a.min(c))
        a.min(b).max(a.min(c)) = a.min(c)
        a.min(b.max(c)) = a.min(c)
        a.min(b.max(c)) = a.min(b).max(a.min(c))
    } else {
        b >= c
        max_eq_left_of_gte(b, c)
        b.max(c) = b
        min_lte_min_right(a, c, b)
        a.min(c) <= a.min(b)
        max_eq_left_of_gte(a.min(b), a.min(c))
        a.min(b).max(a.min(c)) = a.min(b)
        a.min(b.max(c)) = a.min(b)
        a.min(b.max(c)) = a.min(b).max(a.min(c))
    }
}

/// Maximum distributes over minimum on the left.
theorem max_min_distrib_left[L: LinearOrder](a: L, b: L, c: L) {
    a.max(b.min(c)) = a.max(b).min(a.max(c))
} by {
    if b <= c {
        min_eq_left_of_lte(b, c)
        b.min(c) = b
        max_lte_max_right(a, b, c)
        a.max(b) <= a.max(c)
        min_eq_left_of_lte(a.max(b), a.max(c))
        a.max(b).min(a.max(c)) = a.max(b)
        a.max(b.min(c)) = a.max(b)
        a.max(b.min(c)) = a.max(b).min(a.max(c))
    } else {
        b >= c
        min_eq_right_of_gte(b, c)
        b.min(c) = c
        max_lte_max_right(a, c, b)
        a.max(c) <= a.max(b)
        min_eq_right_of_gte(a.max(b), a.max(c))
        a.max(b).min(a.max(c)) = a.max(c)
        a.max(b.min(c)) = a.max(c)
        a.max(b.min(c)) = a.max(b).min(a.max(c))
    }
}
