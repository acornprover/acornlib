/// Public interface for order theory.

from data.basic.relation_basic import is_reflexive, is_transitive, is_antisymmetric
from data.basic.functions import compose, identity_fn, is_injective_fn, is_surjective_fn
from lte import LTE

// base.ac
/// A partial order is a relation that is reflexive, transitive, and antisymmetric.
/// Not all elements need to be comparable.
typeclass P: PartialOrder extends LTE {
    /// The order relation must be reflexive: every element is `≤` itself.
    reflexive {
        is_reflexive(P.lte)
    }

    /// The order relation must be transitive: if `a ≤ b` and `b ≤ c`, then `a ≤ c`.
    transitive {
        is_transitive(P.lte)
    }

    /// The order relation must be antisymmetric: if `a ≤ b` and `b ≤ a`, then `a = b`.
    antisymmetric {
        is_antisymmetric(P.lte)
    }
}

attributes P: PartialOrder {
    /// Strict less-than comparison.
    define lt(self, other: P) -> Bool {
        self <= other and self != other
    }

    /// Greater-than-or-equal-to comparison.
    define gte(self, other: P) -> Bool {
        other <= self
    }

    /// Strict greater-than comparison.
    define gt(self, other: P) -> Bool {
        other < self
    }
}

theorem lte_refl[P: PartialOrder](a: P) {
    a <= a
}

theorem gte_refl[P: PartialOrder](a: P) {
    a >= a
}

/// Every element is below itself.
theorem lte_ref[P: PartialOrder](a: P) {
    a <= a
}

/// Every element is below itself.
theorem lte_self[P: PartialOrder](a: P) {
    a <= a
}

/// Every element is above itself.
theorem gte_self[P: PartialOrder](a: P) {
    a >= a
}

/// Every element is below itself.
theorem le_refl[P: PartialOrder](a: P) {
    a <= a
}

/// Every element is above itself.
theorem ge_refl[P: PartialOrder](a: P) {
    a >= a
}

theorem lte_trans[P: PartialOrder](a: P, b: P, c: P) {
    a <= b and b <= c implies a <= c
}

theorem gte_trans[P: PartialOrder](a: P, b: P, c: P) {
    a >= b and b >= c implies a >= c
}

/// The non-strict order is transitive.
theorem le_trans[P: PartialOrder](a: P, b: P, c: P) {
    a <= b and b <= c implies a <= c
}

/// The reverse non-strict order is transitive.
theorem ge_trans[P: PartialOrder](a: P, b: P, c: P) {
    a >= b and b >= c implies a >= c
}

theorem lte_antisymm[P: PartialOrder](a: P, b: P) {
    a <= b and b <= a implies a = b
}

/// Mutual non-strict bounds force equality.
theorem lte_both_ways_imp_eq[P: PartialOrder](a: P, b: P) {
    a <= b and b <= a implies a = b
}

/// Mutual non-strict bounds force equality.
theorem le_antisymm[P: PartialOrder](a: P, b: P) {
    a <= b and b <= a implies a = b
}

/// Mutual `<=` forces equality on a partial order.
theorem lte_and_gte_imp_eq[P: PartialOrder](a: P, b: P) {
    a <= b and a >= b implies a = b
}

/// A lower and an upper bound force equality.
theorem eq_of_le_of_ge[P: PartialOrder](a: P, b: P) {
    a <= b and a >= b implies a = b
}

/// An upper and a lower bound force equality.
theorem eq_of_ge_of_le[P: PartialOrder](a: P, b: P) {
    a >= b and a <= b implies a = b
}

/// Two partial-order elements are equal when their `<=`-profile agrees on every test point.
theorem eq_of_forall_lte_iff[P: PartialOrder](a: P, b: P) {
    (forall(c: P) { a <= c = b <= c }) implies a = b
}

/// Two partial-order elements are equal when their reverse `<=`-profile agrees on every test point.
theorem eq_of_forall_lte_iff_swap[P: PartialOrder](a: P, b: P) {
    (forall(c: P) { c <= a = c <= b }) implies a = b
}

theorem eq_imp_lte[P: PartialOrder](a: P, b: P) {
    a = b implies a <= b
}

theorem eq_imp_gte[P: PartialOrder](a: P, b: P) {
    a = b implies a >= b
}

/// Equality gives the corresponding non-strict lower bound.
theorem lte_of_eq[P: PartialOrder](a: P, b: P) {
    a = b implies a <= b
}

/// Equality gives the corresponding non-strict lower bound.
theorem le_of_eq[P: PartialOrder](a: P, b: P) {
    a = b implies a <= b
}

/// Equality gives the corresponding non-strict upper bound.
theorem gte_of_eq[P: PartialOrder](a: P, b: P) {
    a = b implies a >= b
}

/// Equality gives the corresponding non-strict upper bound.
theorem ge_of_eq[P: PartialOrder](a: P, b: P) {
    a = b implies a >= b
}

/// Two elements are equal exactly when they bound each other.
theorem eq_iff_lte_and_gte[P: PartialOrder](a: P, b: P) {
    a = b = (a <= b and a >= b)
}

/// Two elements are equal exactly when they bound each other in reverse order.
theorem eq_iff_gte_and_lte[P: PartialOrder](a: P, b: P) {
    a = b = (a >= b and a <= b)
}

/// Distinct elements cannot bound each other.
theorem ne_imp_not_lte_or_not_gte[P: PartialOrder](a: P, b: P) {
    a != b implies not (a <= b) or not (a >= b)
}

/// Distinct elements cannot bound each other, with the disjunction reversed.
theorem ne_imp_not_gte_or_not_lte[P: PartialOrder](a: P, b: P) {
    a != b implies not (a >= b) or not (a <= b)
}

/// Disequality is equivalent to the failure of at least one comparison.
theorem ne_iff_not_lte_or_not_gte[P: PartialOrder](a: P, b: P) {
    a != b = (not (a <= b) or not (a >= b))
}

/// Disequality is equivalent to the reversed failure disjunction.
theorem ne_iff_not_gte_or_not_lte[P: PartialOrder](a: P, b: P) {
    a != b = (not (a >= b) or not (a <= b))
}

theorem lt_imp_lte[P: PartialOrder](a: P, b: P) {
    a < b implies a <= b
}

/// Strict comparison gives the corresponding non-strict comparison.
theorem lte_of_lt[P: PartialOrder](a: P, b: P) {
    a < b implies a <= b
}

/// Strict comparison gives the corresponding non-strict comparison.
theorem le_of_lt[P: PartialOrder](a: P, b: P) {
    a < b implies a <= b
}

theorem gt_imp_gte[P: PartialOrder](a: P, b: P) {
    a > b implies a >= b
}

/// Strict reverse comparison gives the corresponding non-strict reverse comparison.
theorem gte_of_gt[P: PartialOrder](a: P, b: P) {
    a > b implies a >= b
}

/// Strict reverse comparison gives the corresponding non-strict reverse comparison.
theorem ge_of_gt[P: PartialOrder](a: P, b: P) {
    a > b implies a >= b
}

theorem lt_imp_ne[P: PartialOrder](a: P, b: P) {
    a < b implies a != b
}

/// Strict comparison gives disequality.
theorem ne_of_lt[P: PartialOrder](a: P, b: P) {
    a < b implies a != b
}

/// Strict comparison gives inequality.
theorem not_eq_of_lt[P: PartialOrder](a: P, b: P) {
    a < b implies a != b
}

/// Strict comparison gives disequality.
theorem lt_ne[P: PartialOrder](a: P, b: P) {
    a < b implies a != b
}

theorem gt_imp_ne[P: PartialOrder](a: P, b: P) {
    a > b implies a != b
}

/// Strict reverse comparison gives disequality.
theorem ne_of_gt[P: PartialOrder](a: P, b: P) {
    a > b implies a != b
}

/// Strict reverse comparison gives inequality.
theorem not_eq_of_gt[P: PartialOrder](a: P, b: P) {
    a > b implies a != b
}

/// Strict reverse comparison gives disequality.
theorem gt_ne[P: PartialOrder](a: P, b: P) {
    a > b implies a != b
}

theorem lt_imp_ne_symm[P: PartialOrder](a: P, b: P) {
    a < b implies b != a
}

/// Strict comparison gives disequality in the reverse order.
theorem ne_symm_of_lt[P: PartialOrder](a: P, b: P) {
    a < b implies b != a
}

/// Strict comparison gives inequality in the reverse order.
theorem not_eq_symm_of_lt[P: PartialOrder](a: P, b: P) {
    a < b implies b != a
}

theorem gt_imp_ne_symm[P: PartialOrder](a: P, b: P) {
    a > b implies b != a
}

/// Strict reverse comparison gives disequality in the reverse order.
theorem ne_symm_of_gt[P: PartialOrder](a: P, b: P) {
    a > b implies b != a
}

/// Strict reverse comparison gives inequality in the reverse order.
theorem not_eq_symm_of_gt[P: PartialOrder](a: P, b: P) {
    a > b implies b != a
}

theorem not_lt_ref[P: PartialOrder](a: P) {
    not (a < a)
}

/// No element is strictly below itself.
theorem lt_not_ref[P: PartialOrder](a: P) {
    not (a < a)
}

/// No element is strictly below itself.
theorem lt_irrefl[P: PartialOrder](a: P) {
    not (a < a)
}

theorem not_gt_ref[P: PartialOrder](a: P) {
    not (a > a)
}

/// No element is strictly above itself.
theorem gt_irrefl[P: PartialOrder](a: P) {
    not (a > a)
}

/// No element is strictly below itself.
theorem not_lt_self[P: PartialOrder](a: P) {
    not (a < a)
}

/// No element is strictly above itself.
theorem not_gt_self[P: PartialOrder](a: P) {
    not (a > a)
}

theorem lte_imp_not_gt[P: PartialOrder](a: P, b: P) {
    a <= b implies not (a > b)
}

/// A non-strict comparison rules out the opposite strict comparison.
theorem not_gt_of_lte[P: PartialOrder](a: P, b: P) {
    a <= b implies not (a > b)
}

/// A non-strict comparison rules out the opposite strict comparison.
theorem not_gt_of_le[P: PartialOrder](a: P, b: P) {
    a <= b implies not (a > b)
}

theorem gte_imp_not_lt[P: PartialOrder](a: P, b: P) {
    a >= b implies not (a < b)
}

/// A non-strict reverse comparison rules out the strict comparison.
theorem not_lt_of_gte[P: PartialOrder](a: P, b: P) {
    a >= b implies not (a < b)
}

/// A non-strict reverse comparison rules out the strict comparison.
theorem not_lt_of_ge[P: PartialOrder](a: P, b: P) {
    a >= b implies not (a < b)
}

/// A non-strict comparison rules out the swapped strict comparison.
theorem not_lt_swap_of_lte[P: PartialOrder](a: P, b: P) {
    a <= b implies not (b < a)
}

/// A non-strict comparison rules out the swapped strict comparison.
theorem not_lt_swap_of_le[P: PartialOrder](a: P, b: P) {
    a <= b implies not (b < a)
}

/// A non-strict comparison rules out the swapped strict comparison.
theorem lte_imp_not_lt_swap[P: PartialOrder](a: P, b: P) {
    a <= b implies not (b < a)
}

/// A non-strict comparison rules out the swapped strict comparison.
theorem lte_imp_not_lt[P: PartialOrder](a: P, b: P) {
    a <= b implies not (b < a)
}

/// A non-strict comparison rules out the swapped strict comparison.
theorem le_imp_not_lt_swap[P: PartialOrder](a: P, b: P) {
    a <= b implies not (b < a)
}

/// `a > b` rules out `a <= b`.
theorem gt_imp_not_lte[P: PartialOrder](a: P, b: P) {
    a > b implies not (a <= b)
}

/// A strict reverse comparison rules out the non-strict comparison.
theorem not_lte_of_gt[P: PartialOrder](a: P, b: P) {
    a > b implies not (a <= b)
}

/// A strict reverse comparison rules out the non-strict comparison.
theorem not_le_of_gt[P: PartialOrder](a: P, b: P) {
    a > b implies not (a <= b)
}

/// `a < b` rules out `a >= b`.
theorem lt_imp_not_gte[P: PartialOrder](a: P, b: P) {
    a < b implies not (a >= b)
}

/// A strict comparison rules out the non-strict reverse comparison.
theorem not_gte_of_lt[P: PartialOrder](a: P, b: P) {
    a < b implies not (a >= b)
}

/// A strict comparison rules out the non-strict reverse comparison.
theorem not_ge_of_lt[P: PartialOrder](a: P, b: P) {
    a < b implies not (a >= b)
}

/// A strict comparison rules out the swapped strict comparison.
theorem not_lt_both_ways[P: PartialOrder](a: P, b: P) {
    a < b implies not (b < a)
}

/// A strict comparison rules out the swapped strict comparison.
theorem lt_not_symm[P: PartialOrder](a: P, b: P) {
    a < b implies not (b < a)
}

theorem lt_trans[P: PartialOrder](a: P, b: P, c: P) {
    a < b and b < c implies a < c
}

theorem gt_trans[P: PartialOrder](a: P, b: P, c: P) {
    a > b and b > c implies a > c
}

theorem lt_of_lt_of_lte[P: PartialOrder](a: P, b: P, c: P) {
    a < b and b <= c implies a < c
}

theorem lt_of_lte_of_lt[P: PartialOrder](a: P, b: P, c: P) {
    a <= b and b < c implies a < c
}

theorem gt_of_gt_of_gte[P: PartialOrder](a: P, b: P, c: P) {
    a > b and b >= c implies a > c
}

theorem gt_of_gte_of_gt[P: PartialOrder](a: P, b: P, c: P) {
    a >= b and b > c implies a > c
}

/// A strict comparison followed by a non-strict comparison gives a strict comparison.
theorem lt_and_lte[P: PartialOrder](a: P, b: P, c: P) {
    a < b and b <= c implies a < c
}

/// A non-strict comparison followed by a strict comparison gives a strict comparison.
theorem lte_and_lt[P: PartialOrder](a: P, b: P, c: P) {
    a <= b and b < c implies a < c
}

/// A strict comparison followed by a non-strict comparison gives a strict comparison.
theorem lt_lte_trans[P: PartialOrder](a: P, b: P, c: P) {
    a < b and b <= c implies a < c
}

/// A non-strict comparison followed by a strict comparison gives a strict comparison.
theorem lte_lt_trans[P: PartialOrder](a: P, b: P, c: P) {
    a <= b and b < c implies a < c
}

/// A strict reverse comparison followed by a non-strict reverse comparison gives a strict reverse comparison.
theorem gt_and_gte[P: PartialOrder](a: P, b: P, c: P) {
    a > b and b >= c implies a > c
}

/// A non-strict reverse comparison followed by a strict reverse comparison gives a strict reverse comparison.
theorem gte_and_gt[P: PartialOrder](a: P, b: P, c: P) {
    a >= b and b > c implies a > c
}

/// A strict reverse comparison followed by a non-strict reverse comparison gives a strict reverse comparison.
theorem gt_gte_trans[P: PartialOrder](a: P, b: P, c: P) {
    a > b and b >= c implies a > c
}

/// A non-strict reverse comparison followed by a strict reverse comparison gives a strict reverse comparison.
theorem gte_gt_trans[P: PartialOrder](a: P, b: P, c: P) {
    a >= b and b > c implies a > c
}

/// A strict lower bound followed by a non-strict lower bound gives a non-strict lower bound.
theorem lte_of_lt_of_lte[P: PartialOrder](a: P, b: P, c: P) {
    a < b and b <= c implies a <= c
}

/// A non-strict lower bound followed by a strict lower bound gives a non-strict lower bound.
theorem lte_of_lte_of_lt[P: PartialOrder](a: P, b: P, c: P) {
    a <= b and b < c implies a <= c
}

/// A strict upper bound followed by a non-strict upper bound gives a non-strict upper bound.
theorem gte_of_gt_of_gte[P: PartialOrder](a: P, b: P, c: P) {
    a > b and b >= c implies a >= c
}

/// A non-strict upper bound followed by a strict upper bound gives a non-strict upper bound.
theorem gte_of_gte_of_gt[P: PartialOrder](a: P, b: P, c: P) {
    a >= b and b > c implies a >= c
}

/// A strict lower bound followed by a non-strict lower bound gives a non-strict lower bound.
theorem le_of_lt_of_lte[P: PartialOrder](a: P, b: P, c: P) {
    a < b and b <= c implies a <= c
}

/// A non-strict lower bound followed by a strict lower bound gives a non-strict lower bound.
theorem le_of_lte_of_lt[P: PartialOrder](a: P, b: P, c: P) {
    a <= b and b < c implies a <= c
}

/// A strict lower bound followed by a non-strict lower bound gives a strict lower bound.
theorem lt_of_lt_of_le[P: PartialOrder](a: P, b: P, c: P) {
    a < b and b <= c implies a < c
}

/// A non-strict lower bound followed by a strict lower bound gives a strict lower bound.
theorem lt_of_le_of_lt[P: PartialOrder](a: P, b: P, c: P) {
    a <= b and b < c implies a < c
}

/// A strict upper bound followed by a non-strict upper bound gives a non-strict upper bound.
theorem ge_of_gt_of_gte[P: PartialOrder](a: P, b: P, c: P) {
    a > b and b >= c implies a >= c
}

/// A non-strict upper bound followed by a strict upper bound gives a non-strict upper bound.
theorem ge_of_gte_of_gt[P: PartialOrder](a: P, b: P, c: P) {
    a >= b and b > c implies a >= c
}

/// A strict upper bound followed by a non-strict upper bound gives a strict upper bound.
theorem gt_of_gt_of_ge[P: PartialOrder](a: P, b: P, c: P) {
    a > b and b >= c implies a > c
}

/// A non-strict upper bound followed by a strict upper bound gives a strict upper bound.
theorem gt_of_ge_of_gt[P: PartialOrder](a: P, b: P, c: P) {
    a >= b and b > c implies a > c
}

/// Strict order is asymmetric: `a < b` rules out `b < a`.
theorem lt_asymm[P: PartialOrder](a: P, b: P) {
    a < b implies not (b < a)
}

/// Strict greater-than is asymmetric: `a > b` rules out `b > a`.
theorem gt_asymm[P: PartialOrder](a: P, b: P) {
    a > b implies not (b > a)
}

/// A strict comparison rules out the reverse strict comparison.
theorem not_gt_of_lt[P: PartialOrder](a: P, b: P) {
    a < b implies not (a > b)
}

/// A strict reverse comparison rules out the strict comparison.
theorem not_lt_of_gt[P: PartialOrder](a: P, b: P) {
    a > b implies not (a < b)
}

/// A strict comparison rules out the reverse strict comparison.
theorem lt_imp_not_gt[P: PartialOrder](a: P, b: P) {
    a < b implies not (a > b)
}

/// A strict reverse comparison rules out the strict comparison.
theorem gt_imp_not_lt[P: PartialOrder](a: P, b: P) {
    a > b implies not (a < b)
}

/// Replacing the right side of `<=` with an equal element preserves `<=`.
theorem lte_of_lte_of_eq[P: PartialOrder](a: P, b: P, c: P) {
    a <= b and b = c implies a <= c
}

/// Transitivity with equality on the right.
theorem lte_trans_eq[P: PartialOrder](a: P, b: P, c: P) {
    a <= b and b = c implies a <= c
}

/// Replacing the left side of `<=` with an equal element preserves `<=`.
theorem lte_of_eq_of_lte[P: PartialOrder](a: P, b: P, c: P) {
    a = b and b <= c implies a <= c
}

/// Transitivity with equality on the left.
theorem lte_eq_trans[P: PartialOrder](a: P, b: P, c: P) {
    a = b and b <= c implies a <= c
}

/// Replacing the right side of `<=` with an equal element preserves `<=`.
theorem le_of_le_of_eq[P: PartialOrder](a: P, b: P, c: P) {
    a <= b and b = c implies a <= c
}

/// Replacing the left side of `<=` with an equal element preserves `<=`.
theorem le_of_eq_of_le[P: PartialOrder](a: P, b: P, c: P) {
    a = b and b <= c implies a <= c
}

/// Replacing the right side of `<` with an equal element preserves `<`.
theorem lt_of_lt_of_eq[P: PartialOrder](a: P, b: P, c: P) {
    a < b and b = c implies a < c
}

/// Replacing the left side of `<` with an equal element preserves `<`.
theorem lt_of_eq_of_lt[P: PartialOrder](a: P, b: P, c: P) {
    a = b and b < c implies a < c
}

/// Replacing the right side of `>=` with an equal element preserves `>=`.
theorem gte_of_gte_of_eq[P: PartialOrder](a: P, b: P, c: P) {
    a >= b and b = c implies a >= c
}

/// Replacing the left side of `>=` with an equal element preserves `>=`.
theorem gte_of_eq_of_gte[P: PartialOrder](a: P, b: P, c: P) {
    a = b and b >= c implies a >= c
}

/// Replacing the right side of `>=` with an equal element preserves `>=`.
theorem ge_of_ge_of_eq[P: PartialOrder](a: P, b: P, c: P) {
    a >= b and b = c implies a >= c
}

/// Replacing the left side of `>=` with an equal element preserves `>=`.
theorem ge_of_eq_of_ge[P: PartialOrder](a: P, b: P, c: P) {
    a = b and b >= c implies a >= c
}

/// Replacing the right side of `>` with an equal element preserves `>`.
theorem gt_of_gt_of_eq[P: PartialOrder](a: P, b: P, c: P) {
    a > b and b = c implies a > c
}

/// Replacing the left side of `>` with an equal element preserves `>`.
theorem gt_of_eq_of_gt[P: PartialOrder](a: P, b: P, c: P) {
    a = b and b > c implies a > c
}

/// `<` decomposes into `<=` together with inequality.
theorem lt_iff_lte_and_ne[P: PartialOrder](a: P, b: P) {
    a < b = (a <= b and a != b)
}

/// The conjunction `a <= b` and `a != b` is exactly `a < b`.
theorem lte_and_ne_iff_lt[P: PartialOrder](a: P, b: P) {
    (a <= b and a != b) = (a < b)
}

/// `>` decomposes into `>=` together with inequality.
theorem gt_iff_gte_and_ne[P: PartialOrder](a: P, b: P) {
    a > b = (a >= b and a != b)
}

/// The conjunction `a >= b` and `a != b` is exactly `a > b`.
theorem gte_and_ne_iff_gt[P: PartialOrder](a: P, b: P) {
    (a >= b and a != b) = (a > b)
}

/// `<` is the asymmetric part of `<=`: `a < b` iff not `b <= a` and `a <= b`.
theorem lt_iff_lte_not_lte_swap[P: PartialOrder](a: P, b: P) {
    a < b = (not (b <= a) and a <= b)
}

/// `>` is the asymmetric part of `>=`: `a > b` iff not `a <= b` and `b <= a`.
theorem gt_iff_lte_swap_not_lte[P: PartialOrder](a: P, b: P) {
    a > b = (not (a <= b) and b <= a)
}

/// `<=` splits into the disjunction of strict inequality and equality.
theorem lte_iff_lt_or_eq[P: PartialOrder](a: P, b: P) {
    a <= b = (a < b or a = b)
}

/// `<=` splits into the disjunction of equality and strict inequality.
theorem lte_iff_eq_or_lt[P: PartialOrder](a: P, b: P) {
    a <= b = (a = b or a < b)
}

/// `>=` splits into the disjunction of strict greater-than and equality.
theorem gte_iff_gt_or_eq[P: PartialOrder](a: P, b: P) {
    a >= b = (a > b or a = b)
}

/// `>=` splits into the disjunction of equality and strict greater-than.
theorem gte_iff_eq_or_gt[P: PartialOrder](a: P, b: P) {
    a >= b = (a = b or a > b)
}

/// `a <= b` together with `a != b` upgrades to `a < b`.
theorem lt_of_lte_of_ne[P: PartialOrder](a: P, b: P) {
    a <= b and a != b implies a < b
}

/// `a != b` together with `a <= b` upgrades to `a < b`.
theorem lt_of_ne_of_lte[P: PartialOrder](a: P, b: P) {
    a != b and a <= b implies a < b
}

/// `a >= b` together with `a != b` upgrades to `a > b`.
theorem gt_of_gte_of_ne[P: PartialOrder](a: P, b: P) {
    a >= b and a != b implies a > b
}

/// `a != b` together with `a >= b` upgrades to `a > b`.
theorem gt_of_ne_of_gte[P: PartialOrder](a: P, b: P) {
    a != b and a >= b implies a > b
}

/// `a <= b` together with `b != a` upgrades to `a < b`.
theorem lt_of_lte_of_ne_symm[P: PartialOrder](a: P, b: P) {
    a <= b and b != a implies a < b
}

/// `b != a` together with `a <= b` upgrades to `a < b`.
theorem lt_of_ne_symm_of_lte[P: PartialOrder](a: P, b: P) {
    b != a and a <= b implies a < b
}

/// `a >= b` together with `b != a` upgrades to `a > b`.
theorem gt_of_gte_of_ne_symm[P: PartialOrder](a: P, b: P) {
    a >= b and b != a implies a > b
}

/// `b != a` together with `a >= b` upgrades to `a > b`.
theorem gt_of_ne_symm_of_gte[P: PartialOrder](a: P, b: P) {
    b != a and a >= b implies a > b
}

/// On a partial order, `a <= b` either coincides with equality or is strict.
theorem eq_or_lt_of_lte[P: PartialOrder](a: P, b: P) {
    a <= b implies a = b or a < b
}

/// On a partial order, `a <= b` is either strict or equality.
theorem lt_or_eq_of_lte[P: PartialOrder](a: P, b: P) {
    a <= b implies a < b or a = b
}

/// On a partial order, `a >= b` either coincides with equality or is strict.
theorem eq_or_gt_of_gte[P: PartialOrder](a: P, b: P) {
    a >= b implies a = b or a > b
}

/// On a partial order, `a >= b` is either strict or equality.
theorem gt_or_eq_of_gte[P: PartialOrder](a: P, b: P) {
    a >= b implies a > b or a = b
}

/// `<` decomposes into `<=` together with reversed inequality.
theorem lt_iff_lte_and_ne_symm[P: PartialOrder](a: P, b: P) {
    a < b = (a <= b and b != a)
}

/// `>` decomposes into `>=` together with reversed inequality.
theorem gt_iff_gte_and_ne_symm[P: PartialOrder](a: P, b: P) {
    a > b = (a >= b and b != a)
}

/// `<` decomposes into `<=` together with inequality.
theorem lt_iff_le_and_ne[P: PartialOrder](a: P, b: P) {
    a < b = (a <= b and a != b)
}

/// The conjunction `a <= b` and `a != b` is exactly `a < b`.
theorem le_and_ne_iff_lt[P: PartialOrder](a: P, b: P) {
    (a <= b and a != b) = (a < b)
}

/// `>` decomposes into `>=` together with inequality.
theorem gt_iff_ge_and_ne[P: PartialOrder](a: P, b: P) {
    a > b = (a >= b and a != b)
}

/// The conjunction `a >= b` and `a != b` is exactly `a > b`.
theorem ge_and_ne_iff_gt[P: PartialOrder](a: P, b: P) {
    (a >= b and a != b) = (a > b)
}

/// `<` is the asymmetric part of `<=`: `a < b` iff not `b <= a` and `a <= b`.
theorem lt_iff_le_not_le_swap[P: PartialOrder](a: P, b: P) {
    a < b = (not (b <= a) and a <= b)
}

/// `>` is the asymmetric part of `>=`: `a > b` iff not `a <= b` and `b <= a`.
theorem gt_iff_le_swap_not_le[P: PartialOrder](a: P, b: P) {
    a > b = (not (a <= b) and b <= a)
}

/// `<=` splits into the disjunction of strict inequality and equality.
theorem le_iff_lt_or_eq[P: PartialOrder](a: P, b: P) {
    a <= b = (a < b or a = b)
}

/// `<=` splits into the disjunction of equality and strict inequality.
theorem le_iff_eq_or_lt[P: PartialOrder](a: P, b: P) {
    a <= b = (a = b or a < b)
}

/// `>=` splits into the disjunction of strict greater-than and equality.
theorem ge_iff_gt_or_eq[P: PartialOrder](a: P, b: P) {
    a >= b = (a > b or a = b)
}

/// `>=` splits into the disjunction of equality and strict greater-than.
theorem ge_iff_eq_or_gt[P: PartialOrder](a: P, b: P) {
    a >= b = (a = b or a > b)
}

/// `a <= b` together with `a != b` upgrades to `a < b`.
theorem lt_of_le_of_ne[P: PartialOrder](a: P, b: P) {
    a <= b and a != b implies a < b
}

/// `a != b` together with `a <= b` upgrades to `a < b`.
theorem lt_of_ne_of_le[P: PartialOrder](a: P, b: P) {
    a != b and a <= b implies a < b
}

/// `a >= b` together with `a != b` upgrades to `a > b`.
theorem gt_of_ge_of_ne[P: PartialOrder](a: P, b: P) {
    a >= b and a != b implies a > b
}

/// `a != b` together with `a >= b` upgrades to `a > b`.
theorem gt_of_ne_of_ge[P: PartialOrder](a: P, b: P) {
    a != b and a >= b implies a > b
}

/// `a <= b` together with `b != a` upgrades to `a < b`.
theorem lt_of_le_of_ne_symm[P: PartialOrder](a: P, b: P) {
    a <= b and b != a implies a < b
}

/// `b != a` together with `a <= b` upgrades to `a < b`.
theorem lt_of_ne_symm_of_le[P: PartialOrder](a: P, b: P) {
    b != a and a <= b implies a < b
}

/// `a >= b` together with `b != a` upgrades to `a > b`.
theorem gt_of_ge_of_ne_symm[P: PartialOrder](a: P, b: P) {
    a >= b and b != a implies a > b
}

/// `b != a` together with `a >= b` upgrades to `a > b`.
theorem gt_of_ne_symm_of_ge[P: PartialOrder](a: P, b: P) {
    b != a and a >= b implies a > b
}

/// On a partial order, `a <= b` either coincides with equality or is strict.
theorem eq_or_lt_of_le[P: PartialOrder](a: P, b: P) {
    a <= b implies a = b or a < b
}

/// On a partial order, `a <= b` is either strict or equality.
theorem lt_or_eq_of_le[P: PartialOrder](a: P, b: P) {
    a <= b implies a < b or a = b
}

/// On a partial order, `a >= b` either coincides with equality or is strict.
theorem eq_or_gt_of_ge[P: PartialOrder](a: P, b: P) {
    a >= b implies a = b or a > b
}

/// On a partial order, `a >= b` is either strict or equality.
theorem gt_or_eq_of_ge[P: PartialOrder](a: P, b: P) {
    a >= b implies a > b or a = b
}

/// `<` decomposes into `<=` together with reversed inequality.
theorem lt_iff_le_and_ne_symm[P: PartialOrder](a: P, b: P) {
    a < b = (a <= b and b != a)
}

/// `>` decomposes into `>=` together with reversed inequality.
theorem gt_iff_ge_and_ne_symm[P: PartialOrder](a: P, b: P) {
    a > b = (a >= b and b != a)
}

/// A linear order (total order) is a partial order where all elements are comparable.
typeclass L: LinearOrder extends PartialOrder {
    /// All elements are comparable: for any two elements `a` and `b`, either `a ≤ b` or `b ≤ a`.
    totality(a: L, b: L) {
        a <= b or b <= a
    }
}

attributes L: LinearOrder {
    /// Yields the smaller of two elements.
    define min(self, other: L) -> L {
        if self <= other {
            self
        } else {
            other
        }
    }

    /// Yields the larger of two elements.
    define max(self, other: L) -> L {
        if other <= self {
            self
        } else {
            other
        }
    }
}

theorem lte_or_gte[L: LinearOrder](a: L, b: L) {
    a <= b or a >= b
}

/// Any two elements are comparable.
theorem lte_or_lte_swap[L: LinearOrder](a: L, b: L) {
    a <= b or b <= a
}

/// Any two elements are comparable.
theorem le_or_le_swap[L: LinearOrder](a: L, b: L) {
    a <= b or b <= a
}

/// Any two elements are comparable.
theorem le_total[L: LinearOrder](a: L, b: L) {
    a <= b or b <= a
}

/// Any two elements are comparable.
theorem linear_order_total[L: LinearOrder](a: L, b: L) {
    a <= b or b <= a
}

/// Any two elements are comparable.
theorem le_or_ge[L: LinearOrder](a: L, b: L) {
    a <= b or a >= b
}

/// Any two elements are comparable in the reverse disjunction order.
theorem ge_or_le[L: LinearOrder](a: L, b: L) {
    a >= b or a <= b
}

/// Any two elements are comparable in reverse-order form.
theorem gte_or_gte_swap[L: LinearOrder](a: L, b: L) {
    a >= b or b >= a
}

/// Any two elements are comparable in reverse-order form.
theorem ge_or_ge_swap[L: LinearOrder](a: L, b: L) {
    a >= b or b >= a
}

/// Either `a < b` or `b <= a`.
theorem lt_or_lte[L: LinearOrder](a: L, b: L) {
    a < b or b <= a
}

/// Either `a < b` or `b <= a`.
theorem lt_or_le_swap[L: LinearOrder](a: L, b: L) {
    a < b or b <= a
}

/// Either `a <= b` or `b < a`.
theorem lte_or_lt[L: LinearOrder](a: L, b: L) {
    a <= b or b < a
}

/// Either `a <= b` or `b < a`.
theorem le_or_lt_swap[L: LinearOrder](a: L, b: L) {
    a <= b or b < a
}

theorem not_lte_imp_gt[L: LinearOrder](a: L, b: L) {
    not a <= b implies a > b
}

/// Failure of `<=` gives strict greater-than.
theorem gt_of_not_le[L: LinearOrder](a: L, b: L) {
    not (a <= b) implies a > b
}

theorem not_gte_imp_lt[L: LinearOrder](a: L, b: L) {
    not a >= b implies a < b
}

/// Failure of `>=` gives strict less-than.
theorem lt_of_not_ge[L: LinearOrder](a: L, b: L) {
    not (a >= b) implies a < b
}

theorem not_lt_imp_gte[L: LinearOrder](a: L, b: L) {
    not a < b implies a >= b
}

/// Failure of `<` gives non-strict greater-than.
theorem ge_of_not_lt[L: LinearOrder](a: L, b: L) {
    not (a < b) implies a >= b
}

theorem not_gt_imp_lte[L: LinearOrder](a: L, b: L) {
    not a > b implies a <= b
}

/// Failure of `>` gives non-strict less-than.
theorem le_of_not_gt[L: LinearOrder](a: L, b: L) {
    not (a > b) implies a <= b
}

/// Negation of `<=` is exactly `>` on a linear order.
theorem not_lte_iff_gt[L: LinearOrder](a: L, b: L) {
    (not a <= b) = (a > b)
}

/// Negation of `<=` is exactly `>` on a linear order.
theorem not_le_iff_gt[L: LinearOrder](a: L, b: L) {
    (not a <= b) = (a > b)
}

/// Negation of `>=` is exactly `<` on a linear order.
theorem not_gte_iff_lt[L: LinearOrder](a: L, b: L) {
    (not a >= b) = (a < b)
}

/// Negation of `>=` is exactly `<` on a linear order.
theorem not_ge_iff_lt[L: LinearOrder](a: L, b: L) {
    (not a >= b) = (a < b)
}

/// Negation of `<` is exactly `>=` on a linear order.
theorem not_lt_iff_gte[L: LinearOrder](a: L, b: L) {
    (not a < b) = (a >= b)
}

/// Negation of `<` is exactly `>=` on a linear order.
theorem not_lt_iff_ge[L: LinearOrder](a: L, b: L) {
    (not a < b) = (a >= b)
}

/// Negation of `>` is exactly `<=` on a linear order.
theorem not_gt_iff_lte[L: LinearOrder](a: L, b: L) {
    (not a > b) = (a <= b)
}

/// Negation of `>` is exactly `<=` on a linear order.
theorem not_gt_iff_le[L: LinearOrder](a: L, b: L) {
    (not a > b) = (a <= b)
}

theorem lte_or_gt[L: LinearOrder](a: L, b: L) {
    a <= b or a > b
}

/// Either `a <= b` or `a > b`.
theorem le_or_gt[L: LinearOrder](a: L, b: L) {
    a <= b or a > b
}

/// Either `a > b` or `a <= b`.
theorem gt_or_le[L: LinearOrder](a: L, b: L) {
    a > b or a <= b
}

theorem gte_or_lt[L: LinearOrder](a: L, b: L) {
    a >= b or a < b
}

/// Either `a >= b` or `a < b`.
theorem ge_or_lt[L: LinearOrder](a: L, b: L) {
    a >= b or a < b
}

/// Either `a < b` or `a >= b`.
theorem lt_or_ge[L: LinearOrder](a: L, b: L) {
    a < b or a >= b
}

/// Either `a < b` or `a >= b`.
theorem lt_or_gte[L: LinearOrder](a: L, b: L) {
    a < b or a >= b
}

/// Either `a > b` or `a <= b`.
theorem gt_or_lte[L: LinearOrder](a: L, b: L) {
    a > b or a <= b
}

theorem max_imp_gte[L: LinearOrder](a: L, b: L) {
    a.max(b) >= a and a.max(b) >= b
}

theorem min_imp_lte[L: LinearOrder](a: L, b: L) {
    a.min(b) <= a and a.min(b) <= b
}

theorem lt_imp_min[L: LinearOrder](a: L, b: L) {
    a < b implies a.min(b) = a
}

theorem not_lt_imp_min[L: LinearOrder](a: L, b: L) {
    not a < b implies a.min(b) = b
}

theorem gt_imp_min[L: LinearOrder](a: L, b: L) {
    a > b implies a.min(b) = b
}

theorem not_gt_imp_min[L: LinearOrder](a: L, b: L) {
    not a > b implies a.min(b) = a
}

theorem lte_imp_min[L: LinearOrder](a: L, b: L) {
    a <= b implies a.min(b) = a
}

theorem gte_imp_min[L: LinearOrder](a: L, b: L) {
    a >= b implies a.min(b) = b
}

theorem not_gte_imp_min[L: LinearOrder](a: L, b: L) {
    not a >= b implies a.min(b) = a
}

theorem lt_imp_max[L: LinearOrder](a: L, b: L) {
    a < b implies a.max(b) = b
}

theorem not_lt_imp_max[L: LinearOrder](a: L, b: L) {
    not a < b implies a.max(b) = a
}

theorem gt_imp_max[L: LinearOrder](a: L, b: L) {
    a > b implies a.max(b) = a
}

theorem not_gt_imp_max[L: LinearOrder](a: L, b: L) {
    not a > b implies a.max(b) = b
}

theorem gte_imp_max[L: LinearOrder](a: L, b: L) {
    a >= b implies a.max(b) = a
}

theorem not_gte_imp_max[L: LinearOrder](a: L, b: L) {
    not a >= b implies a.max(b) = b
}

theorem min_is_one[L: LinearOrder](a: L, b: L) {
    a.min(b) = a or a.min(b) = b
}

theorem max_is_one[L: LinearOrder](a: L, b: L) {
    a.max(b) = a or a.max(b) = b
}

theorem min_symm[L: LinearOrder](a: L, b: L) {
    a.min(b) = b.min(a)
}

theorem max_symm[L: LinearOrder](a: L, b: L) {
    a.max(b) = b.max(a)
}

/// The minimum operation is commutative.
theorem min_comm[L: LinearOrder](a: L, b: L) {
    a.min(b) = b.min(a)
}

/// The maximum operation is commutative.
theorem max_comm[L: LinearOrder](a: L, b: L) {
    a.max(b) = b.max(a)
}

theorem min_lte_left[L: LinearOrder](a: L, b: L) {
    a.min(b) <= a
}

/// A minimum is below its left argument.
theorem min_le_left[L: LinearOrder](a: L, b: L) {
    a.min(b) <= a
}

theorem min_lte_right[L: LinearOrder](a: L, b: L) {
    a.min(b) <= b
}

/// A minimum is below its right argument.
theorem min_le_right[L: LinearOrder](a: L, b: L) {
    a.min(b) <= b
}

theorem lte_max_left[L: LinearOrder](a: L, b: L) {
    a <= a.max(b)
}

/// The left argument is below the maximum.
theorem le_max_left[L: LinearOrder](a: L, b: L) {
    a <= a.max(b)
}

theorem lte_max_right[L: LinearOrder](a: L, b: L) {
    b <= a.max(b)
}

/// The right argument is below the maximum.
theorem le_max_right[L: LinearOrder](a: L, b: L) {
    b <= a.max(b)
}

/// A minimum is below its left argument, in reverse-order form.
theorem min_gte_left[L: LinearOrder](a: L, b: L) {
    a >= a.min(b)
}

/// A minimum is below its right argument, in reverse-order form.
theorem min_gte_right[L: LinearOrder](a: L, b: L) {
    b >= a.min(b)
}

/// A maximum is above its left argument, in reverse-order form.
theorem max_gte_left[L: LinearOrder](a: L, b: L) {
    a.max(b) >= a
}

/// A maximum is above its right argument, in reverse-order form.
theorem max_gte_right[L: LinearOrder](a: L, b: L) {
    a.max(b) >= b
}

/// A minimum is below its left argument.
theorem min_le_of_left[L: LinearOrder](a: L, b: L) {
    a.min(b) <= a
}

/// A minimum is below its right argument.
theorem min_le_of_right[L: LinearOrder](a: L, b: L) {
    a.min(b) <= b
}

/// The left argument is below the maximum.
theorem le_max_of_left[L: LinearOrder](a: L, b: L) {
    a <= a.max(b)
}

/// The right argument is below the maximum.
theorem le_max_of_right[L: LinearOrder](a: L, b: L) {
    b <= a.max(b)
}

theorem min_idem[L: LinearOrder](a: L) {
    a.min(a) = a
}

theorem max_idem[L: LinearOrder](a: L) {
    a.max(a) = a
}

theorem min_eq_left_of_lte[L: LinearOrder](a: L, b: L) {
    a <= b implies a.min(b) = a
}

theorem min_eq_right_of_gte[L: LinearOrder](a: L, b: L) {
    a >= b implies a.min(b) = b
}

theorem max_eq_right_of_lte[L: LinearOrder](a: L, b: L) {
    a <= b implies a.max(b) = b
}

theorem max_eq_left_of_gte[L: LinearOrder](a: L, b: L) {
    a >= b implies a.max(b) = a
}

/// A strict comparison chooses the left argument as the minimum.
theorem min_eq_left_of_lt[L: LinearOrder](a: L, b: L) {
    a < b implies a.min(b) = a
}

/// A strict reverse comparison chooses the right argument as the minimum.
theorem min_eq_right_of_lt[L: LinearOrder](a: L, b: L) {
    b < a implies a.min(b) = b
}

/// A strict reverse comparison chooses the left argument as the maximum.
theorem max_eq_left_of_gt[L: LinearOrder](a: L, b: L) {
    a > b implies a.max(b) = a
}

/// A strict comparison chooses the right argument as the maximum.
theorem max_eq_right_of_gt[L: LinearOrder](a: L, b: L) {
    b > a implies a.max(b) = b
}

/// A strict comparison chooses the left argument as the minimum.
theorem min_of_lt[L: LinearOrder](a: L, b: L) {
    a < b implies a.min(b) = a
}

/// A strict reverse comparison chooses the right argument as the minimum.
theorem min_of_gt[L: LinearOrder](a: L, b: L) {
    a > b implies a.min(b) = b
}

/// A strict comparison chooses the right argument as the maximum.
theorem max_of_lt[L: LinearOrder](a: L, b: L) {
    a < b implies a.max(b) = b
}

/// A strict reverse comparison chooses the left argument as the maximum.
theorem max_of_gt[L: LinearOrder](a: L, b: L) {
    a > b implies a.max(b) = a
}

/// Failure of a strict reverse comparison chooses the left argument as the minimum.
theorem min_eq_left_of_not_gt[L: LinearOrder](a: L, b: L) {
    not (a > b) implies a.min(b) = a
}

/// Failure of a strict comparison chooses the right argument as the minimum.
theorem min_eq_right_of_not_lt[L: LinearOrder](a: L, b: L) {
    not (a < b) implies a.min(b) = b
}

/// Failure of a strict comparison chooses the left argument as the maximum.
theorem max_eq_left_of_not_lt[L: LinearOrder](a: L, b: L) {
    not (a < b) implies a.max(b) = a
}

/// Failure of a strict reverse comparison chooses the right argument as the maximum.
theorem max_eq_right_of_not_gt[L: LinearOrder](a: L, b: L) {
    not (a > b) implies a.max(b) = b
}

theorem min_absorb_max[L: LinearOrder](a: L, b: L) {
    a.min(a.max(b)) = a
}

theorem max_absorb_min[L: LinearOrder](a: L, b: L) {
    a.max(a.min(b)) = a
}

theorem min_eq_left_iff_lte[L: LinearOrder](a: L, b: L) {
    a.min(b) = a = (a <= b)
}

/// The minimum is the left argument exactly when the left argument is below the right.
theorem min_eq_left_iff_le[L: LinearOrder](a: L, b: L) {
    a.min(b) = a = (a <= b)
}

theorem min_eq_right_iff_gte[L: LinearOrder](a: L, b: L) {
    a.min(b) = b = (a >= b)
}

/// The minimum is the right argument exactly when the left argument is above the right.
theorem min_eq_right_iff_ge[L: LinearOrder](a: L, b: L) {
    a.min(b) = b = (a >= b)
}

theorem max_eq_right_iff_lte[L: LinearOrder](a: L, b: L) {
    a.max(b) = b = (a <= b)
}

/// The maximum is the right argument exactly when the left argument is below the right.
theorem max_eq_right_iff_le[L: LinearOrder](a: L, b: L) {
    a.max(b) = b = (a <= b)
}

theorem max_eq_left_iff_gte[L: LinearOrder](a: L, b: L) {
    a.max(b) = a = (a >= b)
}

/// The maximum is the left argument exactly when the left argument is above the right.
theorem max_eq_left_iff_ge[L: LinearOrder](a: L, b: L) {
    a.max(b) = a = (a >= b)
}

/// A value is below a minimum exactly when it is below both arguments.
theorem lt_min_imp[L: LinearOrder](c: L, a: L, b: L) {
    c < a.min(b) implies c < a and c < b
}

/// Two strict upper bounds of a value give a strict upper bound by their minimum.
theorem lt_min_of_bounds[L: LinearOrder](c: L, a: L, b: L) {
    c < a and c < b implies c < a.min(b)
}

/// A value is below a minimum exactly when it is below both arguments.
theorem lt_min_iff[L: LinearOrder](c: L, a: L, b: L) {
    c < a.min(b) = (c < a and c < b)
}

/// A strict lower bound of both arguments is a strict lower bound of their minimum.
theorem lt_both_imp_lt_min[L: LinearOrder](a: L, b: L, c: L) {
    a < b and a < c implies a < b.min(c)
}

/// A strict lower bound of a minimum is a strict lower bound of the left argument.
theorem lt_min_imp_lt_left[L: LinearOrder](a: L, b: L, c: L) {
    a < b.min(c) implies a < b
}

/// A strict lower bound of a minimum is a strict lower bound of the right argument.
theorem lt_min_imp_lt_right[L: LinearOrder](a: L, b: L, c: L) {
    a < b.min(c) implies a < c
}

/// A maximum is below a value exactly when both arguments are below that value.
theorem max_lt_imp[L: LinearOrder](a: L, b: L, c: L) {
    a.max(b) < c implies a < c and b < c
}

/// Two strict lower bounds of a value give a strict lower bound by their maximum.
theorem max_lt_of_upper_bounds[L: LinearOrder](a: L, b: L, c: L) {
    a < c and b < c implies a.max(b) < c
}

/// A maximum is below a value exactly when both arguments are below that value.
theorem max_lt_iff[L: LinearOrder](a: L, b: L, c: L) {
    a.max(b) < c = (a < c and b < c)
}

/// A strict upper bound of both arguments is a strict upper bound of their maximum.
theorem gt_both_imp_gt_max[L: LinearOrder](a: L, b: L, c: L) {
    a > b and a > c implies a > b.max(c)
}

/// A strict upper bound of a maximum is a strict upper bound of the left argument.
theorem gt_max_imp_gt_left[L: LinearOrder](a: L, b: L, c: L) {
    a > b.max(c) implies a > b
}

/// A strict upper bound of a maximum is a strict upper bound of the right argument.
theorem gt_max_imp_gt_right[L: LinearOrder](a: L, b: L, c: L) {
    a > b.max(c) implies a > c
}

/// A minimum is below a value exactly when at least one argument is below that value.
theorem min_lt_iff[L: LinearOrder](a: L, b: L, c: L) {
    a.min(b) < c = (a < c or b < c)
}

/// If the left argument is below a value, then the minimum is below that value.
theorem min_lt_of_left[L: LinearOrder](a: L, b: L, c: L) {
    a < c implies a.min(b) < c
}

/// If the right argument is below a value, then the minimum is below that value.
theorem min_lt_of_right[L: LinearOrder](a: L, b: L, c: L) {
    b < c implies a.min(b) < c
}

/// If a minimum is below a value, then one argument is below that value.
theorem min_lt_imp_left_or_right[L: LinearOrder](a: L, b: L, c: L) {
    a.min(b) < c implies a < c or b < c
}

/// If one argument is below a value, then the minimum is below that value.
theorem min_lt_of_left_or_right[L: LinearOrder](a: L, b: L, c: L) {
    a < c or b < c implies a.min(b) < c
}

/// A value is below a maximum exactly when it is below at least one argument.
theorem lt_max_iff[L: LinearOrder](c: L, a: L, b: L) {
    c < a.max(b) = (c < a or c < b)
}

/// If a value is below the left argument, then it is below the maximum.
theorem lt_max_of_left[L: LinearOrder](c: L, a: L, b: L) {
    c < a implies c < a.max(b)
}

/// If a value is below the right argument, then it is below the maximum.
theorem lt_max_of_right[L: LinearOrder](c: L, a: L, b: L) {
    c < b implies c < a.max(b)
}

/// If a value is below a maximum, then it is below one argument.
theorem lt_max_imp_left_or_right[L: LinearOrder](c: L, a: L, b: L) {
    c < a.max(b) implies c < a or c < b
}

/// If a value is below one argument, then it is below the maximum.
theorem lt_max_of_left_or_right[L: LinearOrder](c: L, a: L, b: L) {
    c < a or c < b implies c < a.max(b)
}

/// A value is below a minimum only if it is below both arguments.
theorem lte_min_imp[L: LinearOrder](c: L, a: L, b: L) {
    c <= a.min(b) implies c <= a and c <= b
}

/// Two upper bounds of a value give an upper bound by their minimum.
theorem lte_min_of_bounds[L: LinearOrder](c: L, a: L, b: L) {
    c <= a and c <= b implies c <= a.min(b)
}

/// A value is below a minimum exactly when it is below both arguments.
theorem lte_min_iff[L: LinearOrder](c: L, a: L, b: L) {
    c <= a.min(b) = (c <= a and c <= b)
}

/// A value is below a minimum exactly when it is below both arguments.
theorem le_min_iff[L: LinearOrder](c: L, a: L, b: L) {
    c <= a.min(b) = (c <= a and c <= b)
}

/// A lower bound of both arguments is a lower bound of their minimum.
theorem le_min_of_le_left_of_le_right[L: LinearOrder](c: L, a: L, b: L) {
    c <= a and c <= b implies c <= a.min(b)
}

/// A lower bound of a minimum is a lower bound of the left argument.
theorem le_left_of_le_min[L: LinearOrder](c: L, a: L, b: L) {
    c <= a.min(b) implies c <= a
}

/// A lower bound of a minimum is a lower bound of the right argument.
theorem le_right_of_le_min[L: LinearOrder](c: L, a: L, b: L) {
    c <= a.min(b) implies c <= b
}

/// If the left argument is below a value, then the minimum is below that value.
theorem min_lte_of_left[L: LinearOrder](a: L, b: L, c: L) {
    a <= c implies a.min(b) <= c
}

/// If the right argument is below a value, then the minimum is below that value.
theorem min_lte_of_right[L: LinearOrder](a: L, b: L, c: L) {
    b <= c implies a.min(b) <= c
}

/// If a minimum is below a value, then one argument is below that value.
theorem min_lte_imp_left_or_right[L: LinearOrder](a: L, b: L, c: L) {
    a.min(b) <= c implies a <= c or b <= c
}

/// A minimum is above a value only if both arguments are above that value.
theorem min_gte_imp[L: LinearOrder](a: L, b: L, c: L) {
    a.min(b) >= c implies a >= c and b >= c
}

/// Two lower bounds of arguments give a lower bound of their minimum.
theorem min_gte_of_bounds[L: LinearOrder](a: L, b: L, c: L) {
    a >= c and b >= c implies a.min(b) >= c
}

/// A minimum is above a value exactly when both arguments are above that value.
theorem min_gte_iff[L: LinearOrder](a: L, b: L, c: L) {
    a.min(b) >= c = (a >= c and b >= c)
}

/// A minimum is above a value exactly when both arguments are above that value.
theorem min_ge_iff[L: LinearOrder](a: L, b: L, c: L) {
    a.min(b) >= c = (a >= c and b >= c)
}

/// Two upper bounds of a value give an upper bound by their minimum.
theorem min_ge_of_ge_left_of_ge_right[L: LinearOrder](a: L, b: L, c: L) {
    a >= c and b >= c implies a.min(b) >= c
}

/// If a minimum is above a value, then the left argument is above that value.
theorem ge_left_of_min_ge[L: LinearOrder](a: L, b: L, c: L) {
    a.min(b) >= c implies a >= c
}

/// If a minimum is above a value, then the right argument is above that value.
theorem ge_right_of_min_ge[L: LinearOrder](a: L, b: L, c: L) {
    a.min(b) >= c implies b >= c
}

/// A maximum is below a value only if both arguments are below that value.
theorem max_lte_imp[L: LinearOrder](a: L, b: L, c: L) {
    a.max(b) <= c implies a <= c and b <= c
}

/// Two upper bounds of arguments give an upper bound of their maximum.
theorem max_lte_of_upper_bounds[L: LinearOrder](a: L, b: L, c: L) {
    a <= c and b <= c implies a.max(b) <= c
}

/// A maximum is below a value exactly when both arguments are below that value.
theorem max_lte_iff[L: LinearOrder](a: L, b: L, c: L) {
    a.max(b) <= c = (a <= c and b <= c)
}

/// A maximum is below a value exactly when both arguments are below that value.
theorem max_le_iff[L: LinearOrder](a: L, b: L, c: L) {
    a.max(b) <= c = (a <= c and b <= c)
}

/// Two upper bounds give an upper bound of the maximum.
theorem max_le_of_le_left_of_le_right[L: LinearOrder](a: L, b: L, c: L) {
    a <= c and b <= c implies a.max(b) <= c
}

/// If a maximum is below a value, then the left argument is below that value.
theorem le_left_of_max_le[L: LinearOrder](a: L, b: L, c: L) {
    a.max(b) <= c implies a <= c
}

/// If a maximum is below a value, then the right argument is below that value.
theorem le_right_of_max_le[L: LinearOrder](a: L, b: L, c: L) {
    a.max(b) <= c implies b <= c
}

/// If a value is below the left argument, then it is below the maximum.
theorem lte_max_of_left[L: LinearOrder](c: L, a: L, b: L) {
    c <= a implies c <= a.max(b)
}

/// If a value is below the right argument, then it is below the maximum.
theorem lte_max_of_right[L: LinearOrder](c: L, a: L, b: L) {
    c <= b implies c <= a.max(b)
}

/// If a value is below a maximum, then it is below one argument.
theorem lte_max_imp_left_or_right[L: LinearOrder](c: L, a: L, b: L) {
    c <= a.max(b) implies c <= a or c <= b
}

/// The minimum operation is monotone in both arguments.
theorem min_lte_min[L: LinearOrder](a: L, b: L, c: L, d: L) {
    a <= b and c <= d implies a.min(c) <= b.min(d)
}

/// The minimum operation is monotone in both arguments.
theorem min_le_min[L: LinearOrder](a: L, b: L, c: L, d: L) {
    a <= b and c <= d implies a.min(c) <= b.min(d)
}

/// The maximum operation is monotone in both arguments.
theorem max_lte_max[L: LinearOrder](a: L, b: L, c: L, d: L) {
    a <= b and c <= d implies a.max(c) <= b.max(d)
}

/// The maximum operation is monotone in both arguments.
theorem max_le_max[L: LinearOrder](a: L, b: L, c: L, d: L) {
    a <= b and c <= d implies a.max(c) <= b.max(d)
}

/// The minimum operation is monotone in the left argument.
theorem min_lte_min_left[L: LinearOrder](a: L, b: L, c: L) {
    a <= b implies a.min(c) <= b.min(c)
}

/// The minimum operation is monotone in the left argument.
theorem min_le_min_left[L: LinearOrder](a: L, b: L, c: L) {
    a <= b implies a.min(c) <= b.min(c)
}

/// The minimum operation is monotone in the right argument.
theorem min_lte_min_right[L: LinearOrder](a: L, b: L, c: L) {
    b <= c implies a.min(b) <= a.min(c)
}

/// The minimum operation is monotone in the right argument.
theorem min_le_min_right[L: LinearOrder](a: L, b: L, c: L) {
    b <= c implies a.min(b) <= a.min(c)
}

/// The maximum operation is monotone in the left argument.
theorem max_lte_max_left[L: LinearOrder](a: L, b: L, c: L) {
    a <= b implies a.max(c) <= b.max(c)
}

/// The maximum operation is monotone in the left argument.
theorem max_le_max_left[L: LinearOrder](a: L, b: L, c: L) {
    a <= b implies a.max(c) <= b.max(c)
}

/// The maximum operation is monotone in the right argument.
theorem max_lte_max_right[L: LinearOrder](a: L, b: L, c: L) {
    b <= c implies a.max(b) <= a.max(c)
}

/// The maximum operation is monotone in the right argument.
theorem max_le_max_right[L: LinearOrder](a: L, b: L, c: L) {
    b <= c implies a.max(b) <= a.max(c)
}

/// The minimum operation is monotone for reverse-order bounds.
theorem min_gte_min[L: LinearOrder](a: L, b: L, c: L, d: L) {
    a >= b and c >= d implies a.min(c) >= b.min(d)
}

/// The minimum operation is monotone for reverse-order bounds.
theorem min_ge_min[L: LinearOrder](a: L, b: L, c: L, d: L) {
    a >= b and c >= d implies a.min(c) >= b.min(d)
}

/// The maximum operation is monotone for reverse-order bounds.
theorem max_gte_max[L: LinearOrder](a: L, b: L, c: L, d: L) {
    a >= b and c >= d implies a.max(c) >= b.max(d)
}

/// The maximum operation is monotone for reverse-order bounds.
theorem max_ge_max[L: LinearOrder](a: L, b: L, c: L, d: L) {
    a >= b and c >= d implies a.max(c) >= b.max(d)
}

/// The minimum operation is monotone in the left argument for reverse-order bounds.
theorem min_ge_min_left[L: LinearOrder](a: L, b: L, c: L) {
    a >= b implies a.min(c) >= b.min(c)
}

/// The minimum operation is monotone in the right argument for reverse-order bounds.
theorem min_ge_min_right[L: LinearOrder](a: L, b: L, c: L) {
    b >= c implies a.min(b) >= a.min(c)
}

/// The maximum operation is monotone in the left argument for reverse-order bounds.
theorem max_ge_max_left[L: LinearOrder](a: L, b: L, c: L) {
    a >= b implies a.max(c) >= b.max(c)
}

/// The maximum operation is monotone in the right argument for reverse-order bounds.
theorem max_ge_max_right[L: LinearOrder](a: L, b: L, c: L) {
    b >= c implies a.max(b) >= a.max(c)
}

/// A minimum is below a value only if at least one argument is below that value.
theorem min_lte_imp[L: LinearOrder](a: L, b: L, c: L) {
    a.min(b) <= c implies a <= c or b <= c
}

/// A lower argument gives a lower minimum.
theorem min_lte_of_left_or_right[L: LinearOrder](a: L, b: L, c: L) {
    a <= c or b <= c implies a.min(b) <= c
}

/// A minimum is below a value exactly when at least one argument is below that value.
theorem min_lte_iff[L: LinearOrder](a: L, b: L, c: L) {
    a.min(b) <= c = (a <= c or b <= c)
}

/// A value is below a maximum only if it is below at least one argument.
theorem lte_max_imp[L: LinearOrder](c: L, a: L, b: L) {
    c <= a.max(b) implies c <= a or c <= b
}

/// A lower bound of one argument gives a lower bound of the maximum.
theorem lte_max_of_left_or_right[L: LinearOrder](c: L, a: L, b: L) {
    c <= a or c <= b implies c <= a.max(b)
}

/// A value is below a maximum exactly when it is below at least one argument.
theorem lte_max_iff[L: LinearOrder](c: L, a: L, b: L) {
    c <= a.max(b) = (c <= a or c <= b)
}

/// A value is below a maximum exactly when it is below at least one argument.
theorem le_max_iff[L: LinearOrder](c: L, a: L, b: L) {
    c <= a.max(b) = (c <= a or c <= b)
}

/// A lower bound of one argument gives a lower bound of the maximum.
theorem le_max_of_le_left[L: LinearOrder](c: L, a: L, b: L) {
    c <= a implies c <= a.max(b)
}

/// A lower bound of one argument gives a lower bound of the maximum.
theorem le_max_of_le_right[L: LinearOrder](c: L, a: L, b: L) {
    c <= b implies c <= a.max(b)
}

/// A maximum is above a value only if at least one argument is above that value.
theorem max_gte_imp[L: LinearOrder](a: L, b: L, c: L) {
    a.max(b) >= c implies a >= c or b >= c
}

/// A lower bound of one argument gives a lower bound of the maximum.
theorem max_gte_of_left_or_right[L: LinearOrder](a: L, b: L, c: L) {
    a >= c or b >= c implies a.max(b) >= c
}

/// A maximum is above a value exactly when at least one argument is above that value.
theorem max_gte_iff[L: LinearOrder](a: L, b: L, c: L) {
    a.max(b) >= c = (a >= c or b >= c)
}

/// A maximum is above a value exactly when one argument is above that value.
theorem max_ge_iff[L: LinearOrder](a: L, b: L, c: L) {
    a.max(b) >= c = (a >= c or b >= c)
}

/// A lower bound of the left argument is a lower bound of the maximum.
theorem max_ge_of_ge_left[L: LinearOrder](a: L, b: L, c: L) {
    a >= c implies a.max(b) >= c
}

/// A lower bound of the right argument is a lower bound of the maximum.
theorem max_ge_of_ge_right[L: LinearOrder](a: L, b: L, c: L) {
    b >= c implies a.max(b) >= c
}

/// Two elements have a common upper bound.
theorem exists_common_upper_bound[L: LinearOrder](a: L, b: L) {
    exists(n: L) {
        a <= n and b <= n
    }
}

/// Two elements have a common lower bound.
theorem exists_common_lower_bound[L: LinearOrder](a: L, b: L) {
    exists(n: L) {
        n <= a and n <= b
    }
}

/// Two elements have a common upper bound, written with reverse comparisons.
theorem exists_common_ge_bound[L: LinearOrder](a: L, b: L) {
    exists(n: L) {
        n >= a and n >= b
    }
}

/// Two elements have a common lower bound, written with reverse comparisons.
theorem exists_common_le_bound[L: LinearOrder](a: L, b: L) {
    exists(n: L) {
        a >= n and b >= n
    }
}

/// Three elements have a common upper bound.
theorem exists_common_upper_bound_three[L: LinearOrder](a: L, b: L, c: L) {
    exists(n: L) {
        a <= n and b <= n and c <= n
    }
}

/// Three elements have a common lower bound.
theorem exists_common_lower_bound_three[L: LinearOrder](a: L, b: L, c: L) {
    exists(n: L) {
        n <= a and n <= b and n <= c
    }
}

/// Three elements have a common upper bound, written with reverse comparisons.
theorem exists_common_ge_bound_three[L: LinearOrder](a: L, b: L, c: L) {
    exists(n: L) {
        n >= a and n >= b and n >= c
    }
}

/// Three elements have a common lower bound, written with reverse comparisons.
theorem exists_common_le_bound_three[L: LinearOrder](a: L, b: L, c: L) {
    exists(n: L) {
        a >= n and b >= n and c >= n
    }
}

/// The minimum operation is associative.
theorem min_assoc[L: LinearOrder](a: L, b: L, c: L) {
    a.min(b.min(c)) = a.min(b).min(c)
}

/// The maximum operation is associative.
theorem max_assoc[L: LinearOrder](a: L, b: L, c: L) {
    a.max(b.max(c)) = a.max(b).max(c)
}

/// The minimum operation is associative in the reversed nesting.
theorem min_assoc_rev[L: LinearOrder](a: L, b: L, c: L) {
    a.min(b).min(c) = a.min(b.min(c))
}

/// The maximum operation is associative in the reversed nesting.
theorem max_assoc_rev[L: LinearOrder](a: L, b: L, c: L) {
    a.max(b).max(c) = a.max(b.max(c))
}

/// The outer left argument of a nested minimum may be exchanged.
theorem min_left_comm[L: LinearOrder](a: L, b: L, c: L) {
    a.min(b.min(c)) = b.min(a.min(c))
}

/// The outer left argument of a nested maximum may be exchanged.
theorem max_left_comm[L: LinearOrder](a: L, b: L, c: L) {
    a.max(b.max(c)) = b.max(a.max(c))
}

/// The two right arguments of an iterated minimum may be exchanged.
theorem min_right_comm[L: LinearOrder](a: L, b: L, c: L) {
    a.min(b).min(c) = a.min(c).min(b)
}

/// The two right arguments of an iterated maximum may be exchanged.
theorem max_right_comm[L: LinearOrder](a: L, b: L, c: L) {
    a.max(b).max(c) = a.max(c).max(b)
}

/// A minimum absorbs a maximum with the same right argument.
theorem min_absorb_max_right[L: LinearOrder](a: L, b: L) {
    a.min(b.max(a)) = a
}

/// A maximum absorbs a minimum with the same right argument.
theorem max_absorb_min_right[L: LinearOrder](a: L, b: L) {
    a.max(b.min(a)) = a
}

/// Minimum distributes over maximum on the left.
theorem min_max_distrib_left[L: LinearOrder](a: L, b: L, c: L) {
    a.min(b.max(c)) = a.min(b).max(a.min(c))
}

/// Maximum distributes over minimum on the left.
theorem max_min_distrib_left[L: LinearOrder](a: L, b: L, c: L) {
    a.max(b.min(c)) = a.max(b).min(a.max(c))
}




// interval.ac
/// True if an element lies in the closed interval with endpoints `a` and `b`.
define closed_interval[L: LinearOrder](a: L, b: L, x: L) -> Bool {
    a <= x and x <= b
}

/// True if an element lies in the open interval with endpoints `a` and `b`.
define open_interval[L: LinearOrder](a: L, b: L, x: L) -> Bool {
    a < x and x < b
}

/// True if an element lies in the interval open on the left and closed on the right.
define left_open_interval[L: LinearOrder](a: L, b: L, x: L) -> Bool {
    a < x and x <= b
}

/// True if an element lies in the interval closed on the left and open on the right.
define right_open_interval[L: LinearOrder](a: L, b: L, x: L) -> Bool {
    a <= x and x < b
}

/// True if `a` is a lower bound for the elements satisfying `p`.
define is_lower_bound[L: LinearOrder](p: L -> Bool, a: L) -> Bool {
    forall(x: L) {
        p(x) implies a <= x
    }
}

/// True if `b` is an upper bound for the elements satisfying `p`.
define is_upper_bound[L: LinearOrder](p: L -> Bool, b: L) -> Bool {
    forall(x: L) {
        p(x) implies x <= b
    }
}

/// True if the elements satisfying `p` lie in the closed interval from `a` to `b`.
define is_bounded_by_interval[L: LinearOrder](p: L -> Bool, a: L, b: L) -> Bool {
    forall(x: L) {
        p(x) implies closed_interval(a, b, x)
    }
}

/// True if the elements satisfying `p` have a lower bound.
define is_bounded_below[L: LinearOrder](p: L -> Bool) -> Bool {
    exists(a: L) {
        is_lower_bound(p, a)
    }
}

/// True if the elements satisfying `p` have an upper bound.
define is_bounded_above[L: LinearOrder](p: L -> Bool) -> Bool {
    exists(b: L) {
        is_upper_bound(p, b)
    }
}

/// True if the elements satisfying `p` have both a lower and an upper bound.
define is_bounded[L: LinearOrder](p: L -> Bool) -> Bool {
    exists(a: L, b: L) {
        is_bounded_by_interval(p, a, b)
    }
}

/// A lower bound applies to any element satisfying the predicate.
theorem lower_bound_step[L: LinearOrder](p: L -> Bool, a: L, x: L) {
    is_lower_bound(p, a) and p(x) implies a <= x
}

/// An upper bound applies to any element satisfying the predicate.
theorem upper_bound_step[L: LinearOrder](p: L -> Bool, b: L, x: L) {
    is_upper_bound(p, b) and p(x) implies x <= b
}

/// Lower and upper bounds determine an interval bound.
theorem bounds_imp_bounded_by_interval[L: LinearOrder](p: L -> Bool, a: L, b: L) {
    is_lower_bound(p, a) and is_upper_bound(p, b) implies is_bounded_by_interval(p, a, b)
}

/// An interval bound determines a lower bound.
theorem bounded_by_interval_imp_lower_bound[L: LinearOrder](p: L -> Bool, a: L, b: L) {
    is_bounded_by_interval(p, a, b) implies is_lower_bound(p, a)
}

/// An interval bound determines an upper bound.
theorem bounded_by_interval_imp_upper_bound[L: LinearOrder](p: L -> Bool, a: L, b: L) {
    is_bounded_by_interval(p, a, b) implies is_upper_bound(p, b)
}

/// A lower bound gives boundedness below.
theorem lower_bound_imp_bounded_below[L: LinearOrder](p: L -> Bool, a: L) {
    is_lower_bound(p, a) implies is_bounded_below(p)
}

/// An upper bound gives boundedness above.
theorem upper_bound_imp_bounded_above[L: LinearOrder](p: L -> Bool, b: L) {
    is_upper_bound(p, b) implies is_bounded_above(p)
}

/// An interval bound gives boundedness.
theorem bounded_by_interval_imp_bounded[L: LinearOrder](p: L -> Bool, a: L, b: L) {
    is_bounded_by_interval(p, a, b) implies is_bounded(p)
}

// maps.ac
/// True if a map preserves the non-strict order.
define is_monotone[A: PartialOrder, B: PartialOrder](f: A -> B) -> Bool {
    forall(x: A, y: A) {
        x <= y implies f(x) <= f(y)
    }
}

/// True if a map reverses the non-strict order.
define is_antitone[A: PartialOrder, B: PartialOrder](f: A -> B) -> Bool {
    forall(x: A, y: A) {
        x <= y implies f(y) <= f(x)
    }
}

/// True if a map preserves strict order.
define is_strict_monotone[A: PartialOrder, B: PartialOrder](f: A -> B) -> Bool {
    forall(x: A, y: A) {
        x < y implies f(x) < f(y)
    }
}

/// True if a map reverses strict order.
define is_strict_antitone[A: PartialOrder, B: PartialOrder](f: A -> B) -> Bool {
    forall(x: A, y: A) {
        x < y implies f(y) < f(x)
    }
}

/// True if a map reflects and preserves the non-strict order.
define is_order_embedding[A: PartialOrder, B: PartialOrder](f: A -> B) -> Bool {
    forall(x: A, y: A) {
        f(x) <= f(y) = (x <= y)
    }
}

/// True if a map is monotone and surjective.
define is_order_surjection[A: PartialOrder, B: PartialOrder](f: A -> B) -> Bool {
    is_monotone(f) and is_surjective_fn(f)
}

/// A monotone map carries an explicitly ordered pair to an ordered pair.
theorem monotone_step[A: PartialOrder, B: PartialOrder](f: A -> B, x: A, y: A) {
    is_monotone(f) and x <= y implies f(x) <= f(y)
}

/// A monotone map carries an ordered pair to an ordered pair.
theorem monotone_apply[A: PartialOrder, B: PartialOrder](f: A -> B, x: A, y: A) {
    is_monotone(f) and x <= y implies f(x) <= f(y)
}

/// A map is monotone when it carries each ordered pair to an ordered pair.
theorem monotone_from_forall[A: PartialOrder, B: PartialOrder](f: A -> B) {
    forall(x: A, y: A) { x <= y implies f(x) <= f(y) } implies is_monotone(f)
}

/// An antitone map reverses an explicitly ordered pair.
theorem antitone_step[A: PartialOrder, B: PartialOrder](f: A -> B, x: A, y: A) {
    is_antitone(f) and x <= y implies f(y) <= f(x)
}

/// An antitone map reverses an ordered pair.
theorem antitone_apply[A: PartialOrder, B: PartialOrder](f: A -> B, x: A, y: A) {
    is_antitone(f) and x <= y implies f(y) <= f(x)
}

/// A map is antitone when it reverses each ordered pair.
theorem antitone_from_forall[A: PartialOrder, B: PartialOrder](f: A -> B) {
    forall(x: A, y: A) { x <= y implies f(y) <= f(x) } implies is_antitone(f)
}

/// A strict monotone map carries an explicitly strictly ordered pair to a strictly ordered pair.
theorem strict_monotone_step[A: PartialOrder, B: PartialOrder](f: A -> B, x: A, y: A) {
    is_strict_monotone(f) and x < y implies f(x) < f(y)
}

/// A strict monotone map carries a strictly ordered pair to a strictly ordered pair.
theorem strict_monotone_apply[A: PartialOrder, B: PartialOrder](f: A -> B, x: A, y: A) {
    is_strict_monotone(f) and x < y implies f(x) < f(y)
}

/// A strict antitone map reverses an explicitly strictly ordered pair.
theorem strict_antitone_step[A: PartialOrder, B: PartialOrder](f: A -> B, x: A, y: A) {
    is_strict_antitone(f) and x < y implies f(y) < f(x)
}

/// A strict antitone map reverses a strictly ordered pair.
theorem strict_antitone_apply[A: PartialOrder, B: PartialOrder](f: A -> B, x: A, y: A) {
    is_strict_antitone(f) and x < y implies f(y) < f(x)
}

/// An order embedding preserves an ordered pair.
theorem order_embedding_apply_le[A: PartialOrder, B: PartialOrder](f: A -> B, x: A, y: A) {
    is_order_embedding(f) and x <= y implies f(x) <= f(y)
}

/// An order embedding reflects an explicitly ordered image pair.
theorem order_embedding_reflects_lte[A: PartialOrder, B: PartialOrder](f: A -> B, x: A, y: A) {
    is_order_embedding(f) and f(x) <= f(y) implies x <= y
}

/// An order embedding reflects an ordered image pair.
theorem order_embedding_reflects_le[A: PartialOrder, B: PartialOrder](f: A -> B, x: A, y: A) {
    is_order_embedding(f) and f(x) <= f(y) implies x <= y
}

/// An order embedding preserves and reflects non-strict order.
theorem order_embedding_le_iff_le[A: PartialOrder, B: PartialOrder](f: A -> B, x: A, y: A) {
    is_order_embedding(f) implies (f(x) <= f(y) = (x <= y))
}

/// An order embedding reflects a reverse ordered image pair.
theorem order_embedding_reflects_ge[A: PartialOrder, B: PartialOrder](f: A -> B, x: A, y: A) {
    is_order_embedding(f) and f(x) >= f(y) implies x >= y
}

/// An order embedding preserves a strictly ordered pair.
theorem order_embedding_apply_lt[A: PartialOrder, B: PartialOrder](f: A -> B, x: A, y: A) {
    is_order_embedding(f) and x < y implies f(x) < f(y)
}

/// An order embedding preserves a reverse ordered pair.
theorem order_embedding_apply_ge[A: PartialOrder, B: PartialOrder](f: A -> B, x: A, y: A) {
    is_order_embedding(f) and x >= y implies f(x) >= f(y)
}

/// An order embedding preserves and reflects reverse non-strict order.
theorem order_embedding_ge_iff_ge[A: PartialOrder, B: PartialOrder](f: A -> B, x: A, y: A) {
    is_order_embedding(f) implies (f(x) >= f(y) = (x >= y))
}

/// An order embedding preserves a reverse strictly ordered pair.
theorem order_embedding_apply_gt[A: PartialOrder, B: PartialOrder](f: A -> B, x: A, y: A) {
    is_order_embedding(f) and x > y implies f(x) > f(y)
}

/// An order embedding reflects an explicitly strict image pair.
theorem order_embedding_reflects_lt[A: PartialOrder, B: PartialOrder](f: A -> B, x: A, y: A) {
    is_order_embedding(f) and f(x) < f(y) implies x < y
}

/// An order embedding preserves and reflects strict order.
theorem order_embedding_lt_iff_lt[A: PartialOrder, B: PartialOrder](f: A -> B, x: A, y: A) {
    is_order_embedding(f) implies (f(x) < f(y) = (x < y))
}

/// An order embedding reflects an explicitly reversed strict image pair.
theorem order_embedding_reflects_gt[A: PartialOrder, B: PartialOrder](f: A -> B, x: A, y: A) {
    is_order_embedding(f) and f(x) > f(y) implies x > y
}

/// An order embedding preserves and reflects reverse strict order.
theorem order_embedding_gt_iff_gt[A: PartialOrder, B: PartialOrder](f: A -> B, x: A, y: A) {
    is_order_embedding(f) implies (f(x) > f(y) = (x > y))
}

/// An order embedding is monotone.
theorem order_embedding_is_monotone[A: PartialOrder, B: PartialOrder](f: A -> B) {
    is_order_embedding(f) implies is_monotone(f)
}

/// An order embedding is strict monotone.
theorem order_embedding_is_strict_monotone[A: PartialOrder, B: PartialOrder](f: A -> B) {
    is_order_embedding(f) implies is_strict_monotone(f)
}

/// A strict monotone map between linear orders preserves non-strict order.
theorem strict_monotone_preserves_le_of_linear_order[A: LinearOrder, B: LinearOrder](
    f: A -> B, x: A, y: A
) {
    is_strict_monotone(f) and x <= y implies f(x) <= f(y)
}

/// A strict monotone map between linear orders reflects non-strict order.
theorem strict_monotone_reflects_le_of_linear_order[A: LinearOrder, B: LinearOrder](
    f: A -> B, x: A, y: A
) {
    is_strict_monotone(f) and f(x) <= f(y) implies x <= y
}

/// A strict monotone map between linear orders is an order embedding.
theorem strict_monotone_is_order_embedding[A: LinearOrder, B: LinearOrder](f: A -> B) {
    is_strict_monotone(f) implies is_order_embedding(f)
}

/// A strict monotone map between linear orders is injective as a function.
theorem strict_monotone_is_injective_fn[A: LinearOrder, B: LinearOrder](f: A -> B) {
    is_strict_monotone(f) implies is_injective_fn(f)
}

/// The identity map is monotone.
theorem identity_is_monotone[A: PartialOrder] {
    is_monotone(identity_fn[A])
}

/// The identity map is monotone.
theorem monotone_id[A: PartialOrder] {
    is_monotone(identity_fn[A])
}

/// The identity map is strict monotone.
theorem identity_is_strict_monotone[A: PartialOrder] {
    is_strict_monotone(identity_fn[A])
}

/// The identity map is strict monotone.
theorem strict_monotone_id[A: PartialOrder] {
    is_strict_monotone(identity_fn[A])
}

/// The identity map is an order embedding.
theorem identity_is_order_embedding[A: PartialOrder] {
    is_order_embedding(identity_fn[A])
}

/// The composite of order embeddings is an order embedding.
theorem order_embedding_compose[A: PartialOrder, B: PartialOrder, C: PartialOrder](g: A -> B, f: B -> C) {
    is_order_embedding(g) and is_order_embedding(f) implies is_order_embedding(compose(f, g))
}

/// An order embedding preserves binary minima.
theorem order_embedding_min_image[A: LinearOrder, B: LinearOrder](f: A -> B, a: A, b: A) {
    is_order_embedding(f) implies f(a.min(b)) = f(a).min(f(b))
}

/// An order embedding preserves binary maxima.
theorem order_embedding_max_image[A: LinearOrder, B: LinearOrder](f: A -> B, a: A, b: A) {
    is_order_embedding(f) implies f(a.max(b)) = f(a).max(f(b))
}

/// An antitone map sends closed intervals into closed intervals between the reversed image endpoints.
theorem antitone_closed_interval_image[A: LinearOrder, B: LinearOrder](f: A -> B, a: A, b: A, x: A) {
    is_antitone(f) and closed_interval(a, b, x) implies closed_interval(f(b), f(a), f(x))
}

/// A strict antitone map sends open intervals into open intervals between the reversed image endpoints.
theorem strict_antitone_open_interval_image[A: LinearOrder, B: LinearOrder](f: A -> B, a: A, b: A, x: A) {
    is_strict_antitone(f) and open_interval(a, b, x) implies open_interval(f(b), f(a), f(x))
}

/// An antitone and strict antitone map sends left-open intervals into right-open intervals.
theorem antitone_left_open_interval_image[A: LinearOrder, B: LinearOrder](f: A -> B, a: A, b: A, x: A) {
    is_antitone(f) and is_strict_antitone(f) and left_open_interval(a, b, x)
    implies right_open_interval(f(b), f(a), f(x))
}

/// An antitone and strict antitone map sends right-open intervals into left-open intervals.
theorem antitone_right_open_interval_image[A: LinearOrder, B: LinearOrder](f: A -> B, a: A, b: A, x: A) {
    is_antitone(f) and is_strict_antitone(f) and right_open_interval(a, b, x)
    implies left_open_interval(f(b), f(a), f(x))
}

/// An order embedding sends closed intervals into closed intervals between image endpoints.
theorem order_embedding_closed_interval_image[A: LinearOrder, B: LinearOrder](f: A -> B, a: A, b: A, x: A) {
    is_order_embedding(f) and closed_interval(a, b, x) implies closed_interval(f(a), f(b), f(x))
}

/// An order embedding sends open intervals into open intervals between image endpoints.
theorem order_embedding_open_interval_image[A: LinearOrder, B: LinearOrder](f: A -> B, a: A, b: A, x: A) {
    is_order_embedding(f) and open_interval(a, b, x) implies open_interval(f(a), f(b), f(x))
}

/// An order embedding sends left-open intervals into left-open intervals between image endpoints.
theorem order_embedding_left_open_interval_image[A: LinearOrder, B: LinearOrder](f: A -> B, a: A, b: A, x: A) {
    is_order_embedding(f) and left_open_interval(a, b, x) implies left_open_interval(f(a), f(b), f(x))
}

/// An order embedding sends right-open intervals into right-open intervals between image endpoints.
theorem order_embedding_right_open_interval_image[A: LinearOrder, B: LinearOrder](f: A -> B, a: A, b: A, x: A) {
    is_order_embedding(f) and right_open_interval(a, b, x) implies right_open_interval(f(a), f(b), f(x))
}

/// An order embedding reflects membership in a closed interval between image endpoints.
theorem order_embedding_closed_interval_preimage[A: LinearOrder, B: LinearOrder](f: A -> B, a: A, b: A, x: A) {
    is_order_embedding(f) and closed_interval(f(a), f(b), f(x)) implies closed_interval(a, b, x)
}

/// An order embedding reflects membership in an open interval between image endpoints.
theorem order_embedding_open_interval_preimage[A: LinearOrder, B: LinearOrder](f: A -> B, a: A, b: A, x: A) {
    is_order_embedding(f) and open_interval(f(a), f(b), f(x)) implies open_interval(a, b, x)
}

/// An order embedding reflects membership in a left-open interval between image endpoints.
theorem order_embedding_left_open_interval_preimage[A: LinearOrder, B: LinearOrder](f: A -> B, a: A, b: A, x: A) {
    is_order_embedding(f) and left_open_interval(f(a), f(b), f(x)) implies left_open_interval(a, b, x)
}

/// An order embedding reflects membership in a right-open interval between image endpoints.
theorem order_embedding_right_open_interval_preimage[A: LinearOrder, B: LinearOrder](f: A -> B, a: A, b: A, x: A) {
    is_order_embedding(f) and right_open_interval(f(a), f(b), f(x)) implies right_open_interval(a, b, x)
}

/// An order embedding identifies closed-interval membership with image closed-interval membership.
theorem order_embedding_closed_interval_iff[A: LinearOrder, B: LinearOrder](f: A -> B, a: A, b: A, x: A) {
    is_order_embedding(f) implies closed_interval(f(a), f(b), f(x)) = closed_interval(a, b, x)
}

/// An order embedding identifies open-interval membership with image open-interval membership.
theorem order_embedding_open_interval_iff[A: LinearOrder, B: LinearOrder](f: A -> B, a: A, b: A, x: A) {
    is_order_embedding(f) implies open_interval(f(a), f(b), f(x)) = open_interval(a, b, x)
}

/// An order embedding identifies left-open interval membership with image left-open interval membership.
theorem order_embedding_left_open_interval_iff[A: LinearOrder, B: LinearOrder](f: A -> B, a: A, b: A, x: A) {
    is_order_embedding(f) implies left_open_interval(f(a), f(b), f(x)) = left_open_interval(a, b, x)
}

/// An order embedding identifies right-open interval membership with image right-open interval membership.
theorem order_embedding_right_open_interval_iff[A: LinearOrder, B: LinearOrder](f: A -> B, a: A, b: A, x: A) {
    is_order_embedding(f) implies right_open_interval(f(a), f(b), f(x)) = right_open_interval(a, b, x)
}
