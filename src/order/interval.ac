/// Interval predicates for linear orders.

from order.base import LinearOrder, lte_trans, lte_antisymm, lt_trans, lt_imp_lte, not_lt_ref,
    gt_imp_not_lte, lt_of_lte_of_lt, lt_of_lt_of_lte, lte_min_of_bounds, lte_max_left,
    lte_max_right, max_eq_left_of_gte, max_eq_right_of_lte, min_eq_left_of_lte,
    min_eq_right_of_gte, min_lte_left, min_lte_right, max_lte_of_upper_bounds,
    max_lt_of_upper_bounds, lt_min_of_bounds,
    min_lte_min, min_lte_min_left, min_lte_min_right, max_lte_max, max_lte_max_left,
    max_lte_max_right

/// True if an element lies in the closed interval with endpoints `a` and `b`.
define closed_interval[L: LinearOrder](a: L, b: L, x: L) -> Bool {
    a <= x and x <= b
}

/// True if an element lies in the open interval with endpoints `a` and `b`.
define open_interval[L: LinearOrder](a: L, b: L, x: L) -> Bool {
    a < x and x < b
}

/// True if an element lies in the interval open on the left and closed on the right.
define left_open_interval[L: LinearOrder](a: L, b: L, x: L) -> Bool {
    a < x and x <= b
}

/// True if an element lies in the interval closed on the left and open on the right.
define right_open_interval[L: LinearOrder](a: L, b: L, x: L) -> Bool {
    a <= x and x < b
}

/// The point obtained by forcing an element into the closed interval with the given endpoints.
define clamp[L: LinearOrder](a: L, b: L, x: L) -> L {
    a.max(x).min(b)
}

/// Membership in a closed interval is the conjunction of the endpoint inequalities.
theorem closed_interval_at[L: LinearOrder](a: L, b: L, x: L) {
    closed_interval(a, b, x) = (a <= x and x <= b)
}

/// Membership in an open interval is the conjunction of the strict endpoint inequalities.
theorem open_interval_at[L: LinearOrder](a: L, b: L, x: L) {
    open_interval(a, b, x) = (a < x and x < b)
}

/// Membership in a left-open interval is the conjunction of the endpoint inequalities.
theorem left_open_interval_at[L: LinearOrder](a: L, b: L, x: L) {
    left_open_interval(a, b, x) = (a < x and x <= b)
}

/// Membership in a right-open interval is the conjunction of the endpoint inequalities.
theorem right_open_interval_at[L: LinearOrder](a: L, b: L, x: L) {
    right_open_interval(a, b, x) = (a <= x and x < b)
}

/// Clamping is defined as the minimum of the upper endpoint and the maximum of the lower endpoint and the element.
theorem clamp_at[L: LinearOrder](a: L, b: L, x: L) {
    clamp(a, b, x) = a.max(x).min(b)
}

/// The endpoint inequalities determine membership in a closed interval.
theorem closed_interval_intro[L: LinearOrder](a: L, b: L, x: L) {
    a <= x and x <= b implies closed_interval(a, b, x)
} by {
    if a <= x and x <= b {
        closed_interval(a, b, x)
    }
}

/// Endpoint inequalities give membership in a closed interval.
theorem closed_interval_of_le_of_le[L: LinearOrder](a: L, b: L, x: L) {
    a <= x and x <= b implies closed_interval(a, b, x)
} by {
    if a <= x and x <= b {
        closed_interval_intro(a, b, x)
        closed_interval(a, b, x)
    }
}

/// The lower endpoint inequality follows from membership in a closed interval.
theorem closed_interval_left[L: LinearOrder](a: L, b: L, x: L) {
    closed_interval(a, b, x) implies a <= x
} by {
    if closed_interval(a, b, x) {
        a <= x
    }
}

/// Membership in a closed interval gives the lower endpoint inequality.
theorem le_of_closed_interval_left[L: LinearOrder](a: L, b: L, x: L) {
    closed_interval(a, b, x) implies a <= x
} by {
    if closed_interval(a, b, x) {
        closed_interval_left(a, b, x)
        a <= x
    }
}

/// The upper endpoint inequality follows from membership in a closed interval.
theorem closed_interval_right[L: LinearOrder](a: L, b: L, x: L) {
    closed_interval(a, b, x) implies x <= b
} by {
    if closed_interval(a, b, x) {
        x <= b
    }
}

/// Membership in a closed interval gives the upper endpoint inequality.
theorem le_of_closed_interval_right[L: LinearOrder](a: L, b: L, x: L) {
    closed_interval(a, b, x) implies x <= b
} by {
    if closed_interval(a, b, x) {
        closed_interval_right(a, b, x)
        x <= b
    }
}

/// A clamp is above the lower endpoint when the endpoints are ordered.
theorem lte_clamp_of_lte_endpoints[L: LinearOrder](a: L, b: L, x: L) {
    a <= b implies a <= clamp(a, b, x)
} by {
    if a <= b {
        lte_max_left(a, x)
        a <= a.max(x)
        lte_min_of_bounds(a, a.max(x), b)
        a <= a.max(x).min(b)
        a <= clamp(a, b, x)
    }
}

/// A clamp is above the lower endpoint when the endpoints are ordered.
theorem le_clamp_of_le_endpoints[L: LinearOrder](a: L, b: L, x: L) {
    a <= b implies a <= clamp(a, b, x)
} by {
    if a <= b {
        lte_clamp_of_lte_endpoints(a, b, x)
        a <= clamp(a, b, x)
    }
}

/// A clamp is below the upper endpoint.
theorem clamp_lte_right[L: LinearOrder](a: L, b: L, x: L) {
    clamp(a, b, x) <= b
} by {
    min_eq_left_of_lte(a.max(x), b)
    if a.max(x) <= b {
        clamp(a, b, x) = a.max(x)
        a.max(x) <= b
        clamp(a, b, x) <= b
    } else {
        min_eq_right_of_gte(a.max(x), b)
        clamp(a, b, x) = b
        clamp(a, b, x) <= b
    }
}

/// A clamp is below the upper endpoint.
theorem clamp_le_right[L: LinearOrder](a: L, b: L, x: L) {
    clamp(a, b, x) <= b
} by {
    clamp_lte_right(a, b, x)
}

/// A clamp belongs to the closed interval when the endpoints are ordered.
theorem clamp_in_closed_interval[L: LinearOrder](a: L, b: L, x: L) {
    a <= b implies closed_interval(a, b, clamp(a, b, x))
} by {
    if a <= b {
        lte_clamp_of_lte_endpoints(a, b, x)
        a <= clamp(a, b, x)
        clamp_lte_right(a, b, x)
        clamp(a, b, x) <= b
        closed_interval(a, b, clamp(a, b, x))
    }
}

/// Clamping fixes elements already in the closed interval.
theorem clamp_eq_self_of_closed_interval[L: LinearOrder](a: L, b: L, x: L) {
    closed_interval(a, b, x) implies clamp(a, b, x) = x
} by {
    if closed_interval(a, b, x) {
        closed_interval_left(a, b, x)
        a <= x
        max_eq_right_of_lte(a, x)
        a.max(x) = x
        closed_interval_right(a, b, x)
        x <= b
        min_eq_left_of_lte(x, b)
        x.min(b) = x
        clamp(a, b, x) = x
    }
}

/// Clamping an element below the lower endpoint gives the lower endpoint when the endpoints are ordered.
theorem clamp_eq_left_of_lte_of_lte[L: LinearOrder](a: L, b: L, x: L) {
    x <= a and a <= b implies clamp(a, b, x) = a
} by {
    if x <= a and a <= b {
        a >= x
        max_eq_left_of_gte(a, x)
        a.max(x) = a
        min_eq_left_of_lte(a, b)
        a.min(b) = a
        clamp(a, b, x) = a
    }
}

/// Clamping an element above the upper endpoint gives the upper endpoint when the endpoints are ordered.
theorem clamp_eq_right_of_lte_of_lte[L: LinearOrder](a: L, b: L, x: L) {
    a <= b and b <= x implies clamp(a, b, x) = b
} by {
    if a <= b and b <= x {
        lte_trans(a, b, x)
        a <= x
        max_eq_right_of_lte(a, x)
        a.max(x) = x
        x >= b
        min_eq_right_of_gte(x, b)
        x.min(b) = b
        clamp(a, b, x) = b
    }
}

/// Clamping an element below the lower endpoint gives the lower endpoint when the endpoints are ordered.
theorem clamp_eq_left_of_lte[L: LinearOrder](a: L, b: L, x: L) {
    x <= a and a <= b implies clamp(a, b, x) = a
} by {
    if x <= a and a <= b {
        clamp_eq_left_of_lte_of_lte(a, b, x)
        clamp(a, b, x) = a
    }
}

/// Clamping an element above the upper endpoint gives the upper endpoint when the endpoints are ordered.
theorem clamp_eq_right_of_lte[L: LinearOrder](a: L, b: L, x: L) {
    a <= b and b <= x implies clamp(a, b, x) = b
} by {
    if a <= b and b <= x {
        clamp_eq_right_of_lte_of_lte(a, b, x)
        clamp(a, b, x) = b
    }
}

/// Clamping fixes an element between ordered endpoints.
theorem clamp_eq_self_of_lte_of_lte[L: LinearOrder](a: L, b: L, x: L) {
    a <= x and x <= b implies clamp(a, b, x) = x
} by {
    if a <= x and x <= b {
        closed_interval_intro(a, b, x)
        closed_interval(a, b, x)
        clamp_eq_self_of_closed_interval(a, b, x)
        clamp(a, b, x) = x
    }
}

/// Clamping is idempotent when the endpoints are ordered.
theorem clamp_idempotent_of_lte_endpoints[L: LinearOrder](a: L, b: L, x: L) {
    a <= b implies clamp(a, b, clamp(a, b, x)) = clamp(a, b, x)
} by {
    if a <= b {
        clamp_in_closed_interval(a, b, x)
        closed_interval(a, b, clamp(a, b, x))
        clamp_eq_self_of_closed_interval(a, b, clamp(a, b, x))
        clamp(a, b, clamp(a, b, x)) = clamp(a, b, x)
    }
}

/// Clamping is monotone in the element being clamped.
theorem clamp_lte_clamp[L: LinearOrder](a: L, b: L, x: L, y: L) {
    x <= y implies clamp(a, b, x) <= clamp(a, b, y)
} by {
    if x <= y {
        max_lte_max_right(a, x, y)
        a.max(x) <= a.max(y)
        min_lte_min_left(a.max(x), a.max(y), b)
        a.max(x).min(b) <= a.max(y).min(b)
        clamp(a, b, x) <= clamp(a, b, y)
    }
}

/// Clamping is monotone in the element being clamped.
theorem clamp_le_clamp[L: LinearOrder](a: L, b: L, x: L, y: L) {
    x <= y implies clamp(a, b, x) <= clamp(a, b, y)
} by {
    if x <= y {
        clamp_lte_clamp(a, b, x, y)
        clamp(a, b, x) <= clamp(a, b, y)
    }
}

/// Clamping is monotone in the lower endpoint.
theorem clamp_lte_clamp_left[L: LinearOrder](a: L, b: L, c: L, x: L) {
    a <= b implies clamp(a, c, x) <= clamp(b, c, x)
} by {
    if a <= b {
        max_lte_max_left(a, b, x)
        a.max(x) <= b.max(x)
        min_lte_min_left(a.max(x), b.max(x), c)
        a.max(x).min(c) <= b.max(x).min(c)
        clamp(a, c, x) <= clamp(b, c, x)
    }
}

/// Clamping is monotone in the lower endpoint.
theorem clamp_le_clamp_left[L: LinearOrder](a: L, b: L, c: L, x: L) {
    a <= b implies clamp(a, c, x) <= clamp(b, c, x)
} by {
    if a <= b {
        clamp_lte_clamp_left(a, b, c, x)
        clamp(a, c, x) <= clamp(b, c, x)
    }
}

/// Clamping is monotone in the upper endpoint.
theorem clamp_lte_clamp_right[L: LinearOrder](a: L, b: L, c: L, x: L) {
    b <= c implies clamp(a, b, x) <= clamp(a, c, x)
} by {
    if b <= c {
        min_lte_min_right(a.max(x), b, c)
        a.max(x).min(b) <= a.max(x).min(c)
        clamp(a, b, x) <= clamp(a, c, x)
    }
}

/// Clamping is monotone in the upper endpoint.
theorem clamp_le_clamp_right[L: LinearOrder](a: L, b: L, c: L, x: L) {
    b <= c implies clamp(a, b, x) <= clamp(a, c, x)
} by {
    if b <= c {
        clamp_lte_clamp_right(a, b, c, x)
        clamp(a, b, x) <= clamp(a, c, x)
    }
}

/// Clamping is monotone in both endpoints and in the element.
theorem clamp_lte_clamp_of_lte[L: LinearOrder](a1: L, a2: L, b1: L, b2: L, x: L, y: L) {
    a1 <= a2 and b1 <= b2 and x <= y implies clamp(a1, b1, x) <= clamp(a2, b2, y)
} by {
    if a1 <= a2 and b1 <= b2 and x <= y {
        max_lte_max(a1, a2, x, y)
        a1.max(x) <= a2.max(y)
        min_lte_min(a1.max(x), a2.max(y), b1, b2)
        a1.max(x).min(b1) <= a2.max(y).min(b2)
        clamp(a1, b1, x) <= clamp(a2, b2, y)
    }
}

/// Clamping is monotone in both endpoints and in the element.
theorem clamp_le_clamp_of_le[L: LinearOrder](a1: L, a2: L, b1: L, b2: L, x: L, y: L) {
    a1 <= a2 and b1 <= b2 and x <= y implies clamp(a1, b1, x) <= clamp(a2, b2, y)
} by {
    if a1 <= a2 and b1 <= b2 and x <= y {
        clamp_lte_clamp_of_lte(a1, a2, b1, b2, x, y)
        clamp(a1, b1, x) <= clamp(a2, b2, y)
    }
}

/// An ordered closed interval is exactly the fixed-point locus of clamping.
theorem closed_interval_iff_clamp_eq_self[L: LinearOrder](a: L, b: L, x: L) {
    a <= b implies (closed_interval(a, b, x) = (clamp(a, b, x) = x))
} by {
    if a <= b {
        if closed_interval(a, b, x) {
            clamp_eq_self_of_closed_interval(a, b, x)
            clamp(a, b, x) = x
        }
        if clamp(a, b, x) = x {
            clamp_in_closed_interval(a, b, x)
            closed_interval(a, b, clamp(a, b, x))
            closed_interval(a, b, x)
        }
        closed_interval(a, b, x) = (clamp(a, b, x) = x)
    }
}

/// The endpoint inequalities determine membership in an open interval.
theorem open_interval_intro[L: LinearOrder](a: L, b: L, x: L) {
    a < x and x < b implies open_interval(a, b, x)
} by {
    if a < x and x < b {
        open_interval(a, b, x)
    }
}

/// Strict endpoint inequalities give membership in an open interval.
theorem open_interval_of_lt_of_lt[L: LinearOrder](a: L, b: L, x: L) {
    a < x and x < b implies open_interval(a, b, x)
} by {
    if a < x and x < b {
        open_interval_intro(a, b, x)
        open_interval(a, b, x)
    }
}

/// The lower strict endpoint inequality follows from membership in an open interval.
theorem open_interval_left[L: LinearOrder](a: L, b: L, x: L) {
    open_interval(a, b, x) implies a < x
} by {
    if open_interval(a, b, x) {
        a < x
    }
}

/// The upper strict endpoint inequality follows from membership in an open interval.
theorem open_interval_right[L: LinearOrder](a: L, b: L, x: L) {
    open_interval(a, b, x) implies x < b
} by {
    if open_interval(a, b, x) {
        x < b
    }
}

/// The endpoint inequalities determine membership in a left-open interval.
theorem left_open_interval_intro[L: LinearOrder](a: L, b: L, x: L) {
    a < x and x <= b implies left_open_interval(a, b, x)
} by {
    if a < x and x <= b {
        left_open_interval(a, b, x)
    }
}

/// Endpoint inequalities give membership in a left-open interval.
theorem left_open_interval_of_lt_of_le[L: LinearOrder](a: L, b: L, x: L) {
    a < x and x <= b implies left_open_interval(a, b, x)
} by {
    if a < x and x <= b {
        left_open_interval_intro(a, b, x)
        left_open_interval(a, b, x)
    }
}

/// The lower strict endpoint inequality follows from membership in a left-open interval.
theorem left_open_interval_left[L: LinearOrder](a: L, b: L, x: L) {
    left_open_interval(a, b, x) implies a < x
} by {
    if left_open_interval(a, b, x) {
        a < x
    }
}

/// The upper endpoint inequality follows from membership in a left-open interval.
theorem left_open_interval_right[L: LinearOrder](a: L, b: L, x: L) {
    left_open_interval(a, b, x) implies x <= b
} by {
    if left_open_interval(a, b, x) {
        x <= b
    }
}

/// The endpoint inequalities determine membership in a right-open interval.
theorem right_open_interval_intro[L: LinearOrder](a: L, b: L, x: L) {
    a <= x and x < b implies right_open_interval(a, b, x)
} by {
    if a <= x and x < b {
        right_open_interval(a, b, x)
    }
}

/// Endpoint inequalities give membership in a right-open interval.
theorem right_open_interval_of_le_of_lt[L: LinearOrder](a: L, b: L, x: L) {
    a <= x and x < b implies right_open_interval(a, b, x)
} by {
    if a <= x and x < b {
        right_open_interval_intro(a, b, x)
        right_open_interval(a, b, x)
    }
}

/// The lower endpoint inequality follows from membership in a right-open interval.
theorem right_open_interval_left[L: LinearOrder](a: L, b: L, x: L) {
    right_open_interval(a, b, x) implies a <= x
} by {
    if right_open_interval(a, b, x) {
        a <= x
    }
}

/// The upper strict endpoint inequality follows from membership in a right-open interval.
theorem right_open_interval_right[L: LinearOrder](a: L, b: L, x: L) {
    right_open_interval(a, b, x) implies x < b
} by {
    if right_open_interval(a, b, x) {
        x < b
    }
}

/// An open interval is contained in the corresponding closed interval.
theorem open_interval_subset_closed_interval[L: LinearOrder](a: L, b: L, x: L) {
    open_interval(a, b, x) implies closed_interval(a, b, x)
} by {
    if open_interval(a, b, x) {
        a < x
        a <= x
        x < b
        x <= b
        closed_interval(a, b, x)
    }
}

/// A left-open interval is contained in the corresponding closed interval.
theorem left_open_interval_subset_closed_interval[L: LinearOrder](a: L, b: L, x: L) {
    left_open_interval(a, b, x) implies closed_interval(a, b, x)
} by {
    if left_open_interval(a, b, x) {
        a < x
        a <= x
        x <= b
        closed_interval(a, b, x)
    }
}

/// A right-open interval is contained in the corresponding closed interval.
theorem right_open_interval_subset_closed_interval[L: LinearOrder](a: L, b: L, x: L) {
    right_open_interval(a, b, x) implies closed_interval(a, b, x)
} by {
    if right_open_interval(a, b, x) {
        a <= x
        x < b
        x <= b
        closed_interval(a, b, x)
    }
}

/// A closed interval has no members when the left endpoint is strictly above the right endpoint.
theorem not_closed_interval_of_gt_endpoints[L: LinearOrder](a: L, b: L, x: L) {
    a > b implies not closed_interval(a, b, x)
} by {
    if a > b {
        if closed_interval(a, b, x) {
            closed_interval_left(a, b, x)
            a <= x
            closed_interval_right(a, b, x)
            x <= b
            lte_trans(a, x, b)
            a <= b
            gt_imp_not_lte(a, b)
            not (a <= b)
            false
        }
    }
}

/// An open interval has no members when the endpoints are not strictly ordered.
theorem not_open_interval_of_not_lt_endpoints[L: LinearOrder](a: L, b: L, x: L) {
    not (a < b) implies not open_interval(a, b, x)
} by {
    if not (a < b) {
        if open_interval(a, b, x) {
            open_interval_left(a, b, x)
            a < x
            open_interval_right(a, b, x)
            x < b
            lt_trans(a, x, b)
            a < b
            false
        }
    }
}

/// A left-open interval has no members when the endpoints are not strictly ordered.
theorem not_left_open_interval_of_not_lt_endpoints[L: LinearOrder](a: L, b: L, x: L) {
    not (a < b) implies not left_open_interval(a, b, x)
} by {
    if not (a < b) {
        if left_open_interval(a, b, x) {
            left_open_interval_left(a, b, x)
            a < x
            left_open_interval_right(a, b, x)
            x <= b
            lt_of_lt_of_lte(a, x, b)
            a < b
            false
        }
    }
}

/// A right-open interval has no members when the endpoints are not strictly ordered.
theorem not_right_open_interval_of_not_lt_endpoints[L: LinearOrder](a: L, b: L, x: L) {
    not (a < b) implies not right_open_interval(a, b, x)
} by {
    if not (a < b) {
        if right_open_interval(a, b, x) {
            right_open_interval_left(a, b, x)
            a <= x
            right_open_interval_right(a, b, x)
            x < b
            lt_of_lte_of_lt(a, x, b)
            a < b
            false
        }
    }
}

/// A closed interval with equal endpoints contains only that endpoint.
theorem closed_interval_eq_endpoint_of_endpoints_eq[L: LinearOrder](a: L, b: L, x: L) {
    a = b and closed_interval(a, b, x) implies x = a
} by {
    if a = b and closed_interval(a, b, x) {
        closed_interval_left(a, b, x)
        a <= x
        closed_interval_right(a, b, x)
        x <= b
        x <= a
        lte_antisymm(a, x)
        a = x
        x = a
    }
}

/// Ordered endpoints give a member of the closed interval.
theorem closed_interval_nonempty_of_lte[L: LinearOrder](a: L, b: L) {
    a <= b implies exists(x: L) {
        closed_interval(a, b, x)
    }
} by {
    if a <= b {
        closed_interval_intro(a, b, a)
        exists(x: L) {
            x = a and closed_interval(a, b, x)
        }
    }
}

/// Strictly ordered endpoints give a member of the left-open interval.
theorem left_open_interval_nonempty_of_lt[L: LinearOrder](a: L, b: L) {
    a < b implies exists(x: L) {
        left_open_interval(a, b, x)
    }
} by {
    if a < b {
        left_open_interval_intro(a, b, b)
        exists(x: L) {
            x = b and left_open_interval(a, b, x)
        }
    }
}

/// Strictly ordered endpoints give a member of the right-open interval.
theorem right_open_interval_nonempty_of_lt[L: LinearOrder](a: L, b: L) {
    a < b implies exists(x: L) {
        right_open_interval(a, b, x)
    }
} by {
    if a < b {
        right_open_interval_intro(a, b, a)
        exists(x: L) {
            x = a and right_open_interval(a, b, x)
        }
    }
}

/// Membership in a closed interval forces the lower endpoint below the upper endpoint.
theorem closed_interval_endpoints_lte[L: LinearOrder](a: L, b: L, x: L) {
    closed_interval(a, b, x) implies a <= b
} by {
    if closed_interval(a, b, x) {
        a <= x
        x <= b
        lte_trans(a, x, b)
        a <= b
    }
}

/// Membership in an open interval forces the lower endpoint strictly below the upper endpoint.
theorem open_interval_endpoints_lt[L: LinearOrder](a: L, b: L, x: L) {
    open_interval(a, b, x) implies a < b
} by {
    if open_interval(a, b, x) {
        a < x
        x < b
        lt_trans(a, x, b)
        a < b
    }
}

/// Membership in a left-open interval forces the lower endpoint strictly below the upper endpoint.
theorem left_open_interval_endpoints_lt[L: LinearOrder](a: L, b: L, x: L) {
    left_open_interval(a, b, x) implies a < b
} by {
    if left_open_interval(a, b, x) {
        a < x
        x <= b
        lt_of_lt_of_lte(a, x, b)
        a < b
    }
}

/// Membership in a right-open interval forces the lower endpoint strictly below the upper endpoint.
theorem right_open_interval_endpoints_lt[L: LinearOrder](a: L, b: L, x: L) {
    right_open_interval(a, b, x) implies a < b
} by {
    if right_open_interval(a, b, x) {
        a <= x
        x < b
        lt_of_lte_of_lt(a, x, b)
        a < b
    }
}

/// An open interval is contained in the corresponding left-open interval.
theorem open_interval_subset_left_open_interval[L: LinearOrder](a: L, b: L, x: L) {
    open_interval(a, b, x) implies left_open_interval(a, b, x)
} by {
    if open_interval(a, b, x) {
        a < x
        x < b
        x <= b
        left_open_interval(a, b, x)
    }
}

/// An open interval is contained in the corresponding right-open interval.
theorem open_interval_subset_right_open_interval[L: LinearOrder](a: L, b: L, x: L) {
    open_interval(a, b, x) implies right_open_interval(a, b, x)
} by {
    if open_interval(a, b, x) {
        a < x
        a <= x
        x < b
        right_open_interval(a, b, x)
    }
}

/// A point is in the degenerate closed interval at itself.
theorem closed_interval_self[L: LinearOrder](a: L) {
    closed_interval(a, a, a)
} by {
    closed_interval_intro(a, a, a)
}

/// A point belongs to the degenerate closed interval exactly at its endpoint.
theorem closed_interval_self_at[L: LinearOrder](a: L, x: L) {
    closed_interval(a, a, x) implies x = a
} by {
    if closed_interval(a, a, x) {
        a <= x
        x <= a
        lte_antisymm(a, x)
        a = x
        x = a
    }
}

/// The left endpoint belongs to a closed interval exactly when the endpoints are ordered.
theorem closed_interval_left_endpoint[L: LinearOrder](a: L, b: L) {
    a <= b implies closed_interval(a, b, a)
} by {
    if a <= b {
        closed_interval_intro(a, b, a)
    }
}

/// The left endpoint belongs to a closed interval exactly when the endpoints are ordered.
theorem closed_interval_left_endpoint_iff[L: LinearOrder](a: L, b: L) {
    closed_interval(a, b, a) = (a <= b)
} by {
    if closed_interval(a, b, a) {
        closed_interval_right(a, b, a)
        a <= b
    }
    if a <= b {
        closed_interval_left_endpoint(a, b)
        closed_interval(a, b, a)
    }
    closed_interval(a, b, a) = (a <= b)
}

/// The right endpoint belongs to a closed interval exactly when the endpoints are ordered.
theorem closed_interval_right_endpoint[L: LinearOrder](a: L, b: L) {
    a <= b implies closed_interval(a, b, b)
} by {
    if a <= b {
        closed_interval_intro(a, b, b)
    }
}

/// The right endpoint belongs to a closed interval exactly when the endpoints are ordered.
theorem closed_interval_right_endpoint_iff[L: LinearOrder](a: L, b: L) {
    closed_interval(a, b, b) = (a <= b)
} by {
    if closed_interval(a, b, b) {
        closed_interval_left(a, b, b)
        a <= b
    }
    if a <= b {
        closed_interval_right_endpoint(a, b)
        closed_interval(a, b, b)
    }
    closed_interval(a, b, b) = (a <= b)
}

/// The left endpoint does not belong to the corresponding open interval.
theorem not_open_interval_left_endpoint[L: LinearOrder](a: L, b: L) {
    not open_interval(a, b, a)
} by {
    if open_interval(a, b, a) {
        open_interval_left(a, b, a)
        a < a
        not_lt_ref(a)
        false
    }
}

/// The right endpoint does not belong to the corresponding open interval.
theorem not_open_interval_right_endpoint[L: LinearOrder](a: L, b: L) {
    not open_interval(a, b, b)
} by {
    if open_interval(a, b, b) {
        open_interval_right(a, b, b)
        b < b
        not_lt_ref(b)
        false
    }
}

/// The left endpoint does not belong to the corresponding left-open interval.
theorem not_left_open_interval_left_endpoint[L: LinearOrder](a: L, b: L) {
    not left_open_interval(a, b, a)
} by {
    if left_open_interval(a, b, a) {
        left_open_interval_left(a, b, a)
        a < a
        not_lt_ref(a)
        false
    }
}

/// The right endpoint does not belong to the corresponding right-open interval.
theorem not_right_open_interval_right_endpoint[L: LinearOrder](a: L, b: L) {
    not right_open_interval(a, b, b)
} by {
    if right_open_interval(a, b, b) {
        right_open_interval_right(a, b, b)
        b < b
        not_lt_ref(b)
        false
    }
}

/// The left endpoint belongs to a right-open interval when it is strictly below the right endpoint.
theorem right_open_interval_left_endpoint[L: LinearOrder](a: L, b: L) {
    a < b implies right_open_interval(a, b, a)
} by {
    if a < b {
        right_open_interval_intro(a, b, a)
    }
}

/// The left endpoint belongs to a right-open interval exactly when it is strictly below the right endpoint.
theorem right_open_interval_left_endpoint_iff[L: LinearOrder](a: L, b: L) {
    right_open_interval(a, b, a) = (a < b)
} by {
    if right_open_interval(a, b, a) {
        right_open_interval_right(a, b, a)
        a < b
    }
    if a < b {
        right_open_interval_left_endpoint(a, b)
        right_open_interval(a, b, a)
    }
    right_open_interval(a, b, a) = (a < b)
}

/// The right endpoint belongs to a left-open interval when it is strictly above the left endpoint.
theorem left_open_interval_right_endpoint[L: LinearOrder](a: L, b: L) {
    a < b implies left_open_interval(a, b, b)
} by {
    if a < b {
        left_open_interval_intro(a, b, b)
    }
}

/// The right endpoint belongs to a left-open interval exactly when it is strictly above the left endpoint.
theorem left_open_interval_right_endpoint_iff[L: LinearOrder](a: L, b: L) {
    left_open_interval(a, b, b) = (a < b)
} by {
    if left_open_interval(a, b, b) {
        left_open_interval_left(a, b, b)
        a < b
    }
    if a < b {
        left_open_interval_right_endpoint(a, b)
        left_open_interval(a, b, b)
    }
    left_open_interval(a, b, b) = (a < b)
}

/// Enlarging the endpoints preserves membership in a closed interval.
theorem closed_interval_mono[L: LinearOrder](a1: L, a2: L, b1: L, b2: L, x: L) {
    a1 <= a2 and b2 <= b1 and closed_interval(a2, b2, x) implies closed_interval(a1, b1, x)
} by {
    if a1 <= a2 and b2 <= b1 and closed_interval(a2, b2, x) {
        a2 <= x
        lte_trans(a1, a2, x)
        a1 <= x
        x <= b2
        lte_trans(x, b2, b1)
        x <= b1
        closed_interval(a1, b1, x)
    }
}

/// A closed interval is contained in another closed interval when its endpoints are inside the larger bounds.
theorem closed_interval_subset_closed_interval[L: LinearOrder](a1: L, b1: L, a2: L, b2: L, x: L) {
    a2 <= a1 and b1 <= b2 and closed_interval(a1, b1, x) implies closed_interval(a2, b2, x)
} by {
    if a2 <= a1 and b1 <= b2 and closed_interval(a1, b1, x) {
        closed_interval_mono(a2, a1, b2, b1, x)
        closed_interval(a2, b2, x)
    }
}

/// Enlarging the endpoints preserves membership in an open interval.
theorem open_interval_mono[L: LinearOrder](a1: L, a2: L, b1: L, b2: L, x: L) {
    a1 <= a2 and b2 <= b1 and open_interval(a2, b2, x) implies open_interval(a1, b1, x)
} by {
    if a1 <= a2 and b2 <= b1 and open_interval(a2, b2, x) {
        a2 < x
        lt_of_lte_of_lt(a1, a2, x)
        a1 < x
        x < b2
        lt_of_lt_of_lte(x, b2, b1)
        x < b1
        open_interval(a1, b1, x)
    }
}

/// An open interval is contained in another open interval when its endpoints are inside the larger bounds.
theorem open_interval_subset_open_interval[L: LinearOrder](a1: L, b1: L, a2: L, b2: L, x: L) {
    a2 <= a1 and b1 <= b2 and open_interval(a1, b1, x) implies open_interval(a2, b2, x)
} by {
    if a2 <= a1 and b1 <= b2 and open_interval(a1, b1, x) {
        open_interval_mono(a2, a1, b2, b1, x)
        open_interval(a2, b2, x)
    }
}

/// Enlarging the endpoints preserves membership in a left-open interval.
theorem left_open_interval_mono[L: LinearOrder](a1: L, a2: L, b1: L, b2: L, x: L) {
    a1 <= a2 and b2 <= b1 and left_open_interval(a2, b2, x) implies left_open_interval(a1, b1, x)
} by {
    if a1 <= a2 and b2 <= b1 and left_open_interval(a2, b2, x) {
        a2 < x
        lt_of_lte_of_lt(a1, a2, x)
        a1 < x
        x <= b2
        lte_trans(x, b2, b1)
        x <= b1
        left_open_interval(a1, b1, x)
    }
}

/// A left-open interval is contained in another left-open interval when its endpoints are inside the larger bounds.
theorem left_open_interval_subset_left_open_interval[L: LinearOrder](a1: L, b1: L, a2: L, b2: L, x: L) {
    a2 <= a1 and b1 <= b2 and left_open_interval(a1, b1, x) implies left_open_interval(a2, b2, x)
} by {
    if a2 <= a1 and b1 <= b2 and left_open_interval(a1, b1, x) {
        left_open_interval_mono(a2, a1, b2, b1, x)
        left_open_interval(a2, b2, x)
    }
}

/// Enlarging the endpoints preserves membership in a right-open interval.
theorem right_open_interval_mono[L: LinearOrder](a1: L, a2: L, b1: L, b2: L, x: L) {
    a1 <= a2 and b2 <= b1 and right_open_interval(a2, b2, x) implies right_open_interval(a1, b1, x)
} by {
    if a1 <= a2 and b2 <= b1 and right_open_interval(a2, b2, x) {
        a2 <= x
        lte_trans(a1, a2, x)
        a1 <= x
        x < b2
        lt_of_lt_of_lte(x, b2, b1)
        x < b1
        right_open_interval(a1, b1, x)
    }
}

/// A right-open interval is contained in another right-open interval when its endpoints are inside the larger bounds.
theorem right_open_interval_subset_right_open_interval[L: LinearOrder](a1: L, b1: L, a2: L, b2: L, x: L) {
    a2 <= a1 and b1 <= b2 and right_open_interval(a1, b1, x) implies right_open_interval(a2, b2, x)
} by {
    if a2 <= a1 and b1 <= b2 and right_open_interval(a1, b1, x) {
        right_open_interval_mono(a2, a1, b2, b1, x)
        right_open_interval(a2, b2, x)
    }
}

/// An open interval is contained in any closed interval with weaker endpoint bounds.
theorem open_interval_subset_closed_interval_of_bounds[L: LinearOrder](a1: L, b1: L, a2: L, b2: L, x: L) {
    a2 <= a1 and b1 <= b2 and open_interval(a1, b1, x) implies closed_interval(a2, b2, x)
} by {
    if a2 <= a1 and b1 <= b2 and open_interval(a1, b1, x) {
        open_interval_left(a1, b1, x)
        a1 < x
        lt_imp_lte(a1, x)
        a1 <= x
        lte_trans(a2, a1, x)
        a2 <= x
        open_interval_right(a1, b1, x)
        x < b1
        lt_imp_lte(x, b1)
        x <= b1
        lte_trans(x, b1, b2)
        x <= b2
        closed_interval(a2, b2, x)
    }
}

/// An open interval is contained in any left-open interval with weaker endpoint bounds.
theorem open_interval_subset_left_open_interval_of_bounds[L: LinearOrder](a1: L, b1: L, a2: L, b2: L, x: L) {
    a2 <= a1 and b1 <= b2 and open_interval(a1, b1, x) implies left_open_interval(a2, b2, x)
} by {
    if a2 <= a1 and b1 <= b2 and open_interval(a1, b1, x) {
        open_interval_left(a1, b1, x)
        a1 < x
        lt_of_lte_of_lt(a2, a1, x)
        a2 < x
        open_interval_right(a1, b1, x)
        x < b1
        lt_imp_lte(x, b1)
        x <= b1
        lte_trans(x, b1, b2)
        x <= b2
        left_open_interval(a2, b2, x)
    }
}

/// An open interval is contained in any right-open interval with weaker endpoint bounds.
theorem open_interval_subset_right_open_interval_of_bounds[L: LinearOrder](a1: L, b1: L, a2: L, b2: L, x: L) {
    a2 <= a1 and b1 <= b2 and open_interval(a1, b1, x) implies right_open_interval(a2, b2, x)
} by {
    if a2 <= a1 and b1 <= b2 and open_interval(a1, b1, x) {
        open_interval_left(a1, b1, x)
        a1 < x
        lt_imp_lte(a1, x)
        a1 <= x
        lte_trans(a2, a1, x)
        a2 <= x
        open_interval_right(a1, b1, x)
        x < b1
        lt_of_lt_of_lte(x, b1, b2)
        x < b2
        right_open_interval(a2, b2, x)
    }
}

/// A left-open interval is contained in any closed interval with weaker endpoint bounds.
theorem left_open_interval_subset_closed_interval_of_bounds[L: LinearOrder](a1: L, b1: L, a2: L, b2: L, x: L) {
    a2 <= a1 and b1 <= b2 and left_open_interval(a1, b1, x) implies closed_interval(a2, b2, x)
} by {
    if a2 <= a1 and b1 <= b2 and left_open_interval(a1, b1, x) {
        left_open_interval_left(a1, b1, x)
        a1 < x
        lt_imp_lte(a1, x)
        a1 <= x
        lte_trans(a2, a1, x)
        a2 <= x
        left_open_interval_right(a1, b1, x)
        x <= b1
        lte_trans(x, b1, b2)
        x <= b2
        closed_interval(a2, b2, x)
    }
}

/// A right-open interval is contained in any closed interval with weaker endpoint bounds.
theorem right_open_interval_subset_closed_interval_of_bounds[L: LinearOrder](a1: L, b1: L, a2: L, b2: L, x: L) {
    a2 <= a1 and b1 <= b2 and right_open_interval(a1, b1, x) implies closed_interval(a2, b2, x)
} by {
    if a2 <= a1 and b1 <= b2 and right_open_interval(a1, b1, x) {
        right_open_interval_left(a1, b1, x)
        a1 <= x
        lte_trans(a2, a1, x)
        a2 <= x
        right_open_interval_right(a1, b1, x)
        x < b1
        lt_imp_lte(x, b1)
        x <= b1
        lte_trans(x, b1, b2)
        x <= b2
        closed_interval(a2, b2, x)
    }
}

/// A left-open interval is contained in a right-open interval with a strictly weaker upper bound.
theorem left_open_interval_subset_right_open_interval_of_upper_strict_bound[L: LinearOrder](
    a1: L, b1: L, a2: L, b2: L, x: L
) {
    a2 <= a1 and b1 < b2 and left_open_interval(a1, b1, x) implies right_open_interval(a2, b2, x)
} by {
    if a2 <= a1 and b1 < b2 and left_open_interval(a1, b1, x) {
        left_open_interval_left(a1, b1, x)
        a1 < x
        lt_imp_lte(a1, x)
        a1 <= x
        lte_trans(a2, a1, x)
        a2 <= x
        left_open_interval_right(a1, b1, x)
        x <= b1
        lt_of_lte_of_lt(x, b1, b2)
        x < b2
        right_open_interval(a2, b2, x)
    }
}

/// A right-open interval is contained in a left-open interval with a strictly weaker lower bound.
theorem right_open_interval_subset_left_open_interval_of_lower_strict_bound[L: LinearOrder](
    a1: L, b1: L, a2: L, b2: L, x: L
) {
    a2 < a1 and b1 <= b2 and right_open_interval(a1, b1, x) implies left_open_interval(a2, b2, x)
} by {
    if a2 < a1 and b1 <= b2 and right_open_interval(a1, b1, x) {
        right_open_interval_left(a1, b1, x)
        a1 <= x
        lt_of_lt_of_lte(a2, a1, x)
        a2 < x
        right_open_interval_right(a1, b1, x)
        x < b1
        lt_imp_lte(x, b1)
        x <= b1
        lte_trans(x, b1, b2)
        x <= b2
        left_open_interval(a2, b2, x)
    }
}

/// Membership in two closed intervals is membership in the interval with joined lower endpoints and met upper endpoints.
theorem closed_interval_intersection_iff[L: LinearOrder](a1: L, b1: L, a2: L, b2: L, x: L) {
    (closed_interval(a1, b1, x) and closed_interval(a2, b2, x)) =
    closed_interval(a1.max(a2), b1.min(b2), x)
} by {
    if closed_interval(a1, b1, x) and closed_interval(a2, b2, x) {
        closed_interval_left(a1, b1, x)
        a1 <= x
        closed_interval_left(a2, b2, x)
        a2 <= x
        max_lte_of_upper_bounds(a1, a2, x)
        a1.max(a2) <= x
        closed_interval_right(a1, b1, x)
        x <= b1
        closed_interval_right(a2, b2, x)
        x <= b2
        lte_min_of_bounds(x, b1, b2)
        x <= b1.min(b2)
        closed_interval(a1.max(a2), b1.min(b2), x)
    }
    if closed_interval(a1.max(a2), b1.min(b2), x) {
        closed_interval_left(a1.max(a2), b1.min(b2), x)
        a1.max(a2) <= x
        lte_max_left(a1, a2)
        a1 <= a1.max(a2)
        lte_trans(a1, a1.max(a2), x)
        a1 <= x
        lte_max_right(a1, a2)
        a2 <= a1.max(a2)
        lte_trans(a2, a1.max(a2), x)
        a2 <= x
        closed_interval_right(a1.max(a2), b1.min(b2), x)
        x <= b1.min(b2)
        min_lte_left(b1, b2)
        b1.min(b2) <= b1
        lte_trans(x, b1.min(b2), b1)
        x <= b1
        min_lte_right(b1, b2)
        b1.min(b2) <= b2
        lte_trans(x, b1.min(b2), b2)
        x <= b2
        closed_interval(a1, b1, x)
        closed_interval(a2, b2, x)
        closed_interval(a1, b1, x) and closed_interval(a2, b2, x)
    }
    (closed_interval(a1, b1, x) and closed_interval(a2, b2, x)) =
    closed_interval(a1.max(a2), b1.min(b2), x)
}

/// Membership in two closed intervals gives membership in their closed interval intersection.
theorem closed_interval_intersection_intro[L: LinearOrder](a1: L, b1: L, a2: L, b2: L, x: L) {
    closed_interval(a1, b1, x) and closed_interval(a2, b2, x) implies
    closed_interval(a1.max(a2), b1.min(b2), x)
} by {
    if closed_interval(a1, b1, x) and closed_interval(a2, b2, x) {
        closed_interval_intersection_iff(a1, b1, a2, b2, x)
        closed_interval(a1.max(a2), b1.min(b2), x)
    }
}

/// Membership in a closed interval intersection gives membership in the left interval.
theorem closed_interval_intersection_left[L: LinearOrder](a1: L, b1: L, a2: L, b2: L, x: L) {
    closed_interval(a1.max(a2), b1.min(b2), x) implies
    closed_interval(a1, b1, x)
} by {
    if closed_interval(a1.max(a2), b1.min(b2), x) {
        closed_interval_intersection_iff(a1, b1, a2, b2, x)
        closed_interval(a1, b1, x)
    }
}

/// Membership in a closed interval intersection gives membership in the right interval.
theorem closed_interval_intersection_right[L: LinearOrder](a1: L, b1: L, a2: L, b2: L, x: L) {
    closed_interval(a1.max(a2), b1.min(b2), x) implies
    closed_interval(a2, b2, x)
} by {
    if closed_interval(a1.max(a2), b1.min(b2), x) {
        closed_interval_intersection_iff(a1, b1, a2, b2, x)
        closed_interval(a2, b2, x)
    }
}

/// Membership in two open intervals is membership in the open interval with joined lower endpoints and met upper endpoints.
theorem open_interval_intersection_iff[L: LinearOrder](a1: L, b1: L, a2: L, b2: L, x: L) {
    (open_interval(a1, b1, x) and open_interval(a2, b2, x)) =
    open_interval(a1.max(a2), b1.min(b2), x)
} by {
    if open_interval(a1, b1, x) and open_interval(a2, b2, x) {
        open_interval_left(a1, b1, x)
        a1 < x
        open_interval_left(a2, b2, x)
        a2 < x
        max_lt_of_upper_bounds(a1, a2, x)
        a1.max(a2) < x
        open_interval_right(a1, b1, x)
        x < b1
        open_interval_right(a2, b2, x)
        x < b2
        lt_min_of_bounds(x, b1, b2)
        x < b1.min(b2)
        open_interval(a1.max(a2), b1.min(b2), x)
    }
    if open_interval(a1.max(a2), b1.min(b2), x) {
        open_interval_left(a1.max(a2), b1.min(b2), x)
        a1.max(a2) < x
        lte_max_left(a1, a2)
        a1 <= a1.max(a2)
        lt_of_lte_of_lt(a1, a1.max(a2), x)
        a1 < x
        lte_max_right(a1, a2)
        a2 <= a1.max(a2)
        lt_of_lte_of_lt(a2, a1.max(a2), x)
        a2 < x
        open_interval_right(a1.max(a2), b1.min(b2), x)
        x < b1.min(b2)
        min_lte_left(b1, b2)
        b1.min(b2) <= b1
        lt_of_lt_of_lte(x, b1.min(b2), b1)
        x < b1
        min_lte_right(b1, b2)
        b1.min(b2) <= b2
        lt_of_lt_of_lte(x, b1.min(b2), b2)
        x < b2
        open_interval(a1, b1, x)
        open_interval(a2, b2, x)
        open_interval(a1, b1, x) and open_interval(a2, b2, x)
    }
    (open_interval(a1, b1, x) and open_interval(a2, b2, x)) =
    open_interval(a1.max(a2), b1.min(b2), x)
}

/// Membership in two open intervals gives membership in their open interval intersection.
theorem open_interval_intersection_intro[L: LinearOrder](a1: L, b1: L, a2: L, b2: L, x: L) {
    open_interval(a1, b1, x) and open_interval(a2, b2, x) implies
    open_interval(a1.max(a2), b1.min(b2), x)
} by {
    if open_interval(a1, b1, x) and open_interval(a2, b2, x) {
        open_interval_intersection_iff(a1, b1, a2, b2, x)
        open_interval(a1.max(a2), b1.min(b2), x)
    }
}

/// Membership in an open interval intersection gives membership in the left interval.
theorem open_interval_intersection_left[L: LinearOrder](a1: L, b1: L, a2: L, b2: L, x: L) {
    open_interval(a1.max(a2), b1.min(b2), x) implies
    open_interval(a1, b1, x)
} by {
    if open_interval(a1.max(a2), b1.min(b2), x) {
        open_interval_intersection_iff(a1, b1, a2, b2, x)
        open_interval(a1, b1, x)
    }
}

/// Membership in an open interval intersection gives membership in the right interval.
theorem open_interval_intersection_right[L: LinearOrder](a1: L, b1: L, a2: L, b2: L, x: L) {
    open_interval(a1.max(a2), b1.min(b2), x) implies
    open_interval(a2, b2, x)
} by {
    if open_interval(a1.max(a2), b1.min(b2), x) {
        open_interval_intersection_iff(a1, b1, a2, b2, x)
        open_interval(a2, b2, x)
    }
}

/// Membership in two left-open intervals is membership in the left-open interval with joined lower endpoints and met upper endpoints.
theorem left_open_interval_intersection_iff[L: LinearOrder](a1: L, b1: L, a2: L, b2: L, x: L) {
    (left_open_interval(a1, b1, x) and left_open_interval(a2, b2, x)) =
    left_open_interval(a1.max(a2), b1.min(b2), x)
} by {
    if left_open_interval(a1, b1, x) and left_open_interval(a2, b2, x) {
        left_open_interval_left(a1, b1, x)
        a1 < x
        left_open_interval_left(a2, b2, x)
        a2 < x
        max_lt_of_upper_bounds(a1, a2, x)
        a1.max(a2) < x
        left_open_interval_right(a1, b1, x)
        x <= b1
        left_open_interval_right(a2, b2, x)
        x <= b2
        lte_min_of_bounds(x, b1, b2)
        x <= b1.min(b2)
        left_open_interval(a1.max(a2), b1.min(b2), x)
    }
    if left_open_interval(a1.max(a2), b1.min(b2), x) {
        left_open_interval_left(a1.max(a2), b1.min(b2), x)
        a1.max(a2) < x
        lte_max_left(a1, a2)
        a1 <= a1.max(a2)
        lt_of_lte_of_lt(a1, a1.max(a2), x)
        a1 < x
        lte_max_right(a1, a2)
        a2 <= a1.max(a2)
        lt_of_lte_of_lt(a2, a1.max(a2), x)
        a2 < x
        left_open_interval_right(a1.max(a2), b1.min(b2), x)
        x <= b1.min(b2)
        min_lte_left(b1, b2)
        b1.min(b2) <= b1
        lte_trans(x, b1.min(b2), b1)
        x <= b1
        min_lte_right(b1, b2)
        b1.min(b2) <= b2
        lte_trans(x, b1.min(b2), b2)
        x <= b2
        left_open_interval(a1, b1, x)
        left_open_interval(a2, b2, x)
        left_open_interval(a1, b1, x) and left_open_interval(a2, b2, x)
    }
    (left_open_interval(a1, b1, x) and left_open_interval(a2, b2, x)) =
    left_open_interval(a1.max(a2), b1.min(b2), x)
}

/// Membership in two left-open intervals gives membership in their left-open interval intersection.
theorem left_open_interval_intersection_intro[L: LinearOrder](a1: L, b1: L, a2: L, b2: L, x: L) {
    left_open_interval(a1, b1, x) and left_open_interval(a2, b2, x) implies
    left_open_interval(a1.max(a2), b1.min(b2), x)
} by {
    if left_open_interval(a1, b1, x) and left_open_interval(a2, b2, x) {
        left_open_interval_intersection_iff(a1, b1, a2, b2, x)
        left_open_interval(a1.max(a2), b1.min(b2), x)
    }
}

/// Membership in a left-open interval intersection gives membership in the left interval.
theorem left_open_interval_intersection_left[L: LinearOrder](a1: L, b1: L, a2: L, b2: L, x: L) {
    left_open_interval(a1.max(a2), b1.min(b2), x) implies
    left_open_interval(a1, b1, x)
} by {
    if left_open_interval(a1.max(a2), b1.min(b2), x) {
        left_open_interval_intersection_iff(a1, b1, a2, b2, x)
        left_open_interval(a1, b1, x)
    }
}

/// Membership in a left-open interval intersection gives membership in the right interval.
theorem left_open_interval_intersection_right[L: LinearOrder](a1: L, b1: L, a2: L, b2: L, x: L) {
    left_open_interval(a1.max(a2), b1.min(b2), x) implies
    left_open_interval(a2, b2, x)
} by {
    if left_open_interval(a1.max(a2), b1.min(b2), x) {
        left_open_interval_intersection_iff(a1, b1, a2, b2, x)
        left_open_interval(a2, b2, x)
    }
}

/// Membership in two right-open intervals is membership in the right-open interval with joined lower endpoints and met upper endpoints.
theorem right_open_interval_intersection_iff[L: LinearOrder](a1: L, b1: L, a2: L, b2: L, x: L) {
    (right_open_interval(a1, b1, x) and right_open_interval(a2, b2, x)) =
    right_open_interval(a1.max(a2), b1.min(b2), x)
} by {
    if right_open_interval(a1, b1, x) and right_open_interval(a2, b2, x) {
        right_open_interval_left(a1, b1, x)
        a1 <= x
        right_open_interval_left(a2, b2, x)
        a2 <= x
        max_lte_of_upper_bounds(a1, a2, x)
        a1.max(a2) <= x
        right_open_interval_right(a1, b1, x)
        x < b1
        right_open_interval_right(a2, b2, x)
        x < b2
        lt_min_of_bounds(x, b1, b2)
        x < b1.min(b2)
        right_open_interval(a1.max(a2), b1.min(b2), x)
    }
    if right_open_interval(a1.max(a2), b1.min(b2), x) {
        right_open_interval_left(a1.max(a2), b1.min(b2), x)
        a1.max(a2) <= x
        lte_max_left(a1, a2)
        a1 <= a1.max(a2)
        lte_trans(a1, a1.max(a2), x)
        a1 <= x
        lte_max_right(a1, a2)
        a2 <= a1.max(a2)
        lte_trans(a2, a1.max(a2), x)
        a2 <= x
        right_open_interval_right(a1.max(a2), b1.min(b2), x)
        x < b1.min(b2)
        min_lte_left(b1, b2)
        b1.min(b2) <= b1
        lt_of_lt_of_lte(x, b1.min(b2), b1)
        x < b1
        min_lte_right(b1, b2)
        b1.min(b2) <= b2
        lt_of_lt_of_lte(x, b1.min(b2), b2)
        x < b2
        right_open_interval(a1, b1, x)
        right_open_interval(a2, b2, x)
        right_open_interval(a1, b1, x) and right_open_interval(a2, b2, x)
    }
    (right_open_interval(a1, b1, x) and right_open_interval(a2, b2, x)) =
    right_open_interval(a1.max(a2), b1.min(b2), x)
}

/// Membership in two right-open intervals gives membership in their right-open interval intersection.
theorem right_open_interval_intersection_intro[L: LinearOrder](a1: L, b1: L, a2: L, b2: L, x: L) {
    right_open_interval(a1, b1, x) and right_open_interval(a2, b2, x) implies
    right_open_interval(a1.max(a2), b1.min(b2), x)
} by {
    if right_open_interval(a1, b1, x) and right_open_interval(a2, b2, x) {
        right_open_interval_intersection_iff(a1, b1, a2, b2, x)
        right_open_interval(a1.max(a2), b1.min(b2), x)
    }
}

/// Membership in a right-open interval intersection gives membership in the left interval.
theorem right_open_interval_intersection_left[L: LinearOrder](a1: L, b1: L, a2: L, b2: L, x: L) {
    right_open_interval(a1.max(a2), b1.min(b2), x) implies
    right_open_interval(a1, b1, x)
} by {
    if right_open_interval(a1.max(a2), b1.min(b2), x) {
        right_open_interval_intersection_iff(a1, b1, a2, b2, x)
        right_open_interval(a1, b1, x)
    }
}

/// Membership in a right-open interval intersection gives membership in the right interval.
theorem right_open_interval_intersection_right[L: LinearOrder](a1: L, b1: L, a2: L, b2: L, x: L) {
    right_open_interval(a1.max(a2), b1.min(b2), x) implies
    right_open_interval(a2, b2, x)
} by {
    if right_open_interval(a1.max(a2), b1.min(b2), x) {
        right_open_interval_intersection_iff(a1, b1, a2, b2, x)
        right_open_interval(a2, b2, x)
    }
}

/// True if `a` is a lower bound for the elements satisfying `p`.
define is_lower_bound[L: LinearOrder](p: L -> Bool, a: L) -> Bool {
    forall(x: L) {
        p(x) implies a <= x
    }
}

/// True if `b` is an upper bound for the elements satisfying `p`.
define is_upper_bound[L: LinearOrder](p: L -> Bool, b: L) -> Bool {
    forall(x: L) {
        p(x) implies x <= b
    }
}

/// True if the elements satisfying `p` lie in the closed interval from `a` to `b`.
define is_bounded_by_interval[L: LinearOrder](p: L -> Bool, a: L, b: L) -> Bool {
    forall(x: L) {
        p(x) implies closed_interval(a, b, x)
    }
}

/// True if the elements satisfying `p` have a lower bound.
define is_bounded_below[L: LinearOrder](p: L -> Bool) -> Bool {
    exists(a: L) {
        is_lower_bound(p, a)
    }
}

/// True if the elements satisfying `p` have an upper bound.
define is_bounded_above[L: LinearOrder](p: L -> Bool) -> Bool {
    exists(b: L) {
        is_upper_bound(p, b)
    }
}

/// True if the elements satisfying `p` have both a lower and an upper bound.
define is_bounded[L: LinearOrder](p: L -> Bool) -> Bool {
    exists(a: L, b: L) {
        is_bounded_by_interval(p, a, b)
    }
}

/// A lower bound applies to any element satisfying the predicate.
theorem lower_bound_step[L: LinearOrder](p: L -> Bool, a: L, x: L) {
    is_lower_bound(p, a) and p(x) implies a <= x
} by {
    if is_lower_bound(p, a) and p(x) {
        is_lower_bound(p, a) = forall(y: L) {
            p(y) implies a <= y
        }
        a <= x
    }
}

/// An upper bound applies to any element satisfying the predicate.
theorem upper_bound_step[L: LinearOrder](p: L -> Bool, b: L, x: L) {
    is_upper_bound(p, b) and p(x) implies x <= b
} by {
    if is_upper_bound(p, b) and p(x) {
        is_upper_bound(p, b) = forall(y: L) {
            p(y) implies y <= b
        }
        x <= b
    }
}

/// An interval bound applies to any element satisfying the predicate.
theorem bounded_by_interval_step[L: LinearOrder](p: L -> Bool, a: L, b: L, x: L) {
    is_bounded_by_interval(p, a, b) and p(x) implies closed_interval(a, b, x)
} by {
    if is_bounded_by_interval(p, a, b) and p(x) {
        is_bounded_by_interval(p, a, b) = forall(y: L) {
            p(y) implies closed_interval(a, b, y)
        }
        closed_interval(a, b, x)
    }
}

/// Lower and upper bounds determine an interval bound.
theorem bounds_imp_bounded_by_interval[L: LinearOrder](p: L -> Bool, a: L, b: L) {
    is_lower_bound(p, a) and is_upper_bound(p, b) implies is_bounded_by_interval(p, a, b)
} by {
    if is_lower_bound(p, a) and is_upper_bound(p, b) {
        forall(x: L) {
            if p(x) {
                lower_bound_step(p, a, x)
                a <= x
                upper_bound_step(p, b, x)
                x <= b
                closed_interval(a, b, x)
            }
        }
    }
}

/// An interval bound determines a lower bound.
theorem bounded_by_interval_imp_lower_bound[L: LinearOrder](p: L -> Bool, a: L, b: L) {
    is_bounded_by_interval(p, a, b) implies is_lower_bound(p, a)
} by {
    if is_bounded_by_interval(p, a, b) {
        forall(x: L) {
            if p(x) {
                bounded_by_interval_step(p, a, b, x)
                closed_interval(a, b, x)
                a <= x
            }
        }
    }
}

/// An interval bound determines an upper bound.
theorem bounded_by_interval_imp_upper_bound[L: LinearOrder](p: L -> Bool, a: L, b: L) {
    is_bounded_by_interval(p, a, b) implies is_upper_bound(p, b)
} by {
    if is_bounded_by_interval(p, a, b) {
        forall(x: L) {
            if p(x) {
                bounded_by_interval_step(p, a, b, x)
                closed_interval(a, b, x)
                x <= b
            }
        }
    }
}

/// Being bounded by an interval is equivalent to having the corresponding lower and upper bounds.
theorem bounded_by_interval_iff_bounds[L: LinearOrder](p: L -> Bool, a: L, b: L) {
    is_bounded_by_interval(p, a, b) = (is_lower_bound(p, a) and is_upper_bound(p, b))
} by {
    if is_bounded_by_interval(p, a, b) {
        bounded_by_interval_imp_lower_bound(p, a, b)
        is_lower_bound(p, a)
        bounded_by_interval_imp_upper_bound(p, a, b)
        is_upper_bound(p, b)
        is_lower_bound(p, a) and is_upper_bound(p, b)
    }
    if is_lower_bound(p, a) and is_upper_bound(p, b) {
        bounds_imp_bounded_by_interval(p, a, b)
        is_bounded_by_interval(p, a, b)
    }
    is_bounded_by_interval(p, a, b) = (is_lower_bound(p, a) and is_upper_bound(p, b))
}

/// The left endpoint is a lower bound for the corresponding closed interval.
theorem closed_interval_lower_bound[L: LinearOrder](a: L, b: L) {
    is_lower_bound(closed_interval(a, b), a)
} by {
    forall(x: L) {
        if closed_interval(a, b, x) {
            closed_interval_left(a, b, x)
            a <= x
        }
    }
}

/// The right endpoint is an upper bound for the corresponding closed interval.
theorem closed_interval_upper_bound[L: LinearOrder](a: L, b: L) {
    is_upper_bound(closed_interval(a, b), b)
} by {
    forall(x: L) {
        if closed_interval(a, b, x) {
            closed_interval_right(a, b, x)
            x <= b
        }
    }
}

/// A closed interval is bounded by its endpoints.
theorem closed_interval_bounded_by_self[L: LinearOrder](a: L, b: L) {
    is_bounded_by_interval(closed_interval(a, b), a, b)
} by {
    closed_interval_lower_bound(a, b)
    closed_interval_upper_bound(a, b)
    bounds_imp_bounded_by_interval(closed_interval(a, b), a, b)
}

/// The left endpoint is a lower bound for the corresponding open interval.
theorem open_interval_lower_bound[L: LinearOrder](a: L, b: L) {
    is_lower_bound(open_interval(a, b), a)
} by {
    forall(x: L) {
        if open_interval(a, b, x) {
            open_interval_left(a, b, x)
            a < x
            lt_imp_lte(a, x)
            a <= x
        }
    }
}

/// The right endpoint is an upper bound for the corresponding open interval.
theorem open_interval_upper_bound[L: LinearOrder](a: L, b: L) {
    is_upper_bound(open_interval(a, b), b)
} by {
    forall(x: L) {
        if open_interval(a, b, x) {
            open_interval_right(a, b, x)
            x < b
            lt_imp_lte(x, b)
            x <= b
        }
    }
}

/// An open interval is bounded by its endpoints.
theorem open_interval_bounded_by_closed_interval[L: LinearOrder](a: L, b: L) {
    is_bounded_by_interval(open_interval(a, b), a, b)
} by {
    open_interval_lower_bound(a, b)
    open_interval_upper_bound(a, b)
    bounds_imp_bounded_by_interval(open_interval(a, b), a, b)
}

/// The left endpoint is a lower bound for the corresponding left-open interval.
theorem left_open_interval_lower_bound[L: LinearOrder](a: L, b: L) {
    is_lower_bound(left_open_interval(a, b), a)
} by {
    forall(x: L) {
        if left_open_interval(a, b, x) {
            left_open_interval_left(a, b, x)
            a < x
            lt_imp_lte(a, x)
            a <= x
        }
    }
}

/// The right endpoint is an upper bound for the corresponding left-open interval.
theorem left_open_interval_upper_bound[L: LinearOrder](a: L, b: L) {
    is_upper_bound(left_open_interval(a, b), b)
} by {
    forall(x: L) {
        if left_open_interval(a, b, x) {
            left_open_interval_right(a, b, x)
            x <= b
        }
    }
}

/// A left-open interval is bounded by its endpoints.
theorem left_open_interval_bounded_by_closed_interval[L: LinearOrder](a: L, b: L) {
    is_bounded_by_interval(left_open_interval(a, b), a, b)
} by {
    left_open_interval_lower_bound(a, b)
    left_open_interval_upper_bound(a, b)
    bounds_imp_bounded_by_interval(left_open_interval(a, b), a, b)
}

/// The left endpoint is a lower bound for the corresponding right-open interval.
theorem right_open_interval_lower_bound[L: LinearOrder](a: L, b: L) {
    is_lower_bound(right_open_interval(a, b), a)
} by {
    forall(x: L) {
        if right_open_interval(a, b, x) {
            right_open_interval_left(a, b, x)
            a <= x
        }
    }
}

/// The right endpoint is an upper bound for the corresponding right-open interval.
theorem right_open_interval_upper_bound[L: LinearOrder](a: L, b: L) {
    is_upper_bound(right_open_interval(a, b), b)
} by {
    forall(x: L) {
        if right_open_interval(a, b, x) {
            right_open_interval_right(a, b, x)
            x < b
            lt_imp_lte(x, b)
            x <= b
        }
    }
}

/// A right-open interval is bounded by its endpoints.
theorem right_open_interval_bounded_by_closed_interval[L: LinearOrder](a: L, b: L) {
    is_bounded_by_interval(right_open_interval(a, b), a, b)
} by {
    right_open_interval_lower_bound(a, b)
    right_open_interval_upper_bound(a, b)
    bounds_imp_bounded_by_interval(right_open_interval(a, b), a, b)
}

/// A closed interval is bounded below.
theorem closed_interval_bounded_below[L: LinearOrder](a: L, b: L) {
    is_bounded_below(closed_interval(a, b))
} by {
    closed_interval_lower_bound(a, b)
    exists(x: L) {
        x = a and is_lower_bound(closed_interval(a, b), x)
    }
    is_bounded_below(closed_interval(a, b))
}

/// A closed interval is bounded above.
theorem closed_interval_bounded_above[L: LinearOrder](a: L, b: L) {
    is_bounded_above(closed_interval(a, b))
} by {
    closed_interval_upper_bound(a, b)
    exists(x: L) {
        x = b and is_upper_bound(closed_interval(a, b), x)
    }
    is_bounded_above(closed_interval(a, b))
}

/// A closed interval is bounded.
theorem closed_interval_bounded[L: LinearOrder](a: L, b: L) {
    is_bounded(closed_interval(a, b))
} by {
    closed_interval_bounded_by_self(a, b)
    exists(x: L, y: L) {
        x = a and y = b and is_bounded_by_interval(closed_interval(a, b), x, y)
    }
    is_bounded(closed_interval(a, b))
}

/// An open interval is bounded below.
theorem open_interval_bounded_below[L: LinearOrder](a: L, b: L) {
    is_bounded_below(open_interval(a, b))
} by {
    open_interval_lower_bound(a, b)
    exists(x: L) {
        x = a and is_lower_bound(open_interval(a, b), x)
    }
    is_bounded_below(open_interval(a, b))
}

/// An open interval is bounded above.
theorem open_interval_bounded_above[L: LinearOrder](a: L, b: L) {
    is_bounded_above(open_interval(a, b))
} by {
    open_interval_upper_bound(a, b)
    exists(x: L) {
        x = b and is_upper_bound(open_interval(a, b), x)
    }
    is_bounded_above(open_interval(a, b))
}

/// An open interval is bounded.
theorem open_interval_bounded[L: LinearOrder](a: L, b: L) {
    is_bounded(open_interval(a, b))
} by {
    open_interval_bounded_by_closed_interval(a, b)
    exists(x: L, y: L) {
        x = a and y = b and is_bounded_by_interval(open_interval(a, b), x, y)
    }
    is_bounded(open_interval(a, b))
}

/// A left-open interval is bounded below.
theorem left_open_interval_bounded_below[L: LinearOrder](a: L, b: L) {
    is_bounded_below(left_open_interval(a, b))
} by {
    left_open_interval_lower_bound(a, b)
    exists(x: L) {
        x = a and is_lower_bound(left_open_interval(a, b), x)
    }
    is_bounded_below(left_open_interval(a, b))
}

/// A left-open interval is bounded above.
theorem left_open_interval_bounded_above[L: LinearOrder](a: L, b: L) {
    is_bounded_above(left_open_interval(a, b))
} by {
    left_open_interval_upper_bound(a, b)
    exists(x: L) {
        x = b and is_upper_bound(left_open_interval(a, b), x)
    }
    is_bounded_above(left_open_interval(a, b))
}

/// A left-open interval is bounded.
theorem left_open_interval_bounded[L: LinearOrder](a: L, b: L) {
    is_bounded(left_open_interval(a, b))
} by {
    left_open_interval_bounded_by_closed_interval(a, b)
    exists(x: L, y: L) {
        x = a and y = b and is_bounded_by_interval(left_open_interval(a, b), x, y)
    }
    is_bounded(left_open_interval(a, b))
}

/// A right-open interval is bounded below.
theorem right_open_interval_bounded_below[L: LinearOrder](a: L, b: L) {
    is_bounded_below(right_open_interval(a, b))
} by {
    right_open_interval_lower_bound(a, b)
    exists(x: L) {
        x = a and is_lower_bound(right_open_interval(a, b), x)
    }
    is_bounded_below(right_open_interval(a, b))
}

/// A right-open interval is bounded above.
theorem right_open_interval_bounded_above[L: LinearOrder](a: L, b: L) {
    is_bounded_above(right_open_interval(a, b))
} by {
    right_open_interval_upper_bound(a, b)
    exists(x: L) {
        x = b and is_upper_bound(right_open_interval(a, b), x)
    }
    is_bounded_above(right_open_interval(a, b))
}

/// A right-open interval is bounded.
theorem right_open_interval_bounded[L: LinearOrder](a: L, b: L) {
    is_bounded(right_open_interval(a, b))
} by {
    right_open_interval_bounded_by_closed_interval(a, b)
    exists(x: L, y: L) {
        x = a and y = b and is_bounded_by_interval(right_open_interval(a, b), x, y)
    }
    is_bounded(right_open_interval(a, b))
}

/// A lower bound remains a lower bound after moving it downward.
theorem lower_bound_mono[L: LinearOrder](p: L -> Bool, a: L, b: L) {
    a <= b and is_lower_bound(p, b) implies is_lower_bound(p, a)
} by {
    if a <= b and is_lower_bound(p, b) {
        forall(x: L) {
            if p(x) {
                lower_bound_step(p, b, x)
                b <= x
                lte_trans(a, b, x)
                a <= x
            }
        }
    }
}

/// An upper bound remains an upper bound after moving it upward.
theorem upper_bound_mono[L: LinearOrder](p: L -> Bool, a: L, b: L) {
    a <= b and is_upper_bound(p, a) implies is_upper_bound(p, b)
} by {
    if a <= b and is_upper_bound(p, a) {
        forall(x: L) {
            if p(x) {
                upper_bound_step(p, a, x)
                x <= a
                lte_trans(x, a, b)
                x <= b
            }
        }
    }
}

/// Enlarging an interval bound preserves boundedness by an interval.
theorem bounded_by_interval_mono[L: LinearOrder](p: L -> Bool, a1: L, a2: L, b1: L, b2: L) {
    a1 <= a2 and b2 <= b1 and is_bounded_by_interval(p, a2, b2) implies is_bounded_by_interval(p, a1, b1)
} by {
    if a1 <= a2 and b2 <= b1 and is_bounded_by_interval(p, a2, b2) {
        bounded_by_interval_imp_lower_bound(p, a2, b2)
        is_lower_bound(p, a2)
        lower_bound_mono(p, a1, a2)
        is_lower_bound(p, a1)
        bounded_by_interval_imp_upper_bound(p, a2, b2)
        is_upper_bound(p, b2)
        upper_bound_mono(p, b2, b1)
        is_upper_bound(p, b1)
        bounds_imp_bounded_by_interval(p, a1, b1)
        is_bounded_by_interval(p, a1, b1)
    }
}

/// A lower bound gives boundedness below.
theorem lower_bound_imp_bounded_below[L: LinearOrder](p: L -> Bool, a: L) {
    is_lower_bound(p, a) implies is_bounded_below(p)
} by {
    if is_lower_bound(p, a) {
        exists(x: L) {
            x = a and is_lower_bound(p, x)
        }
        is_bounded_below(p)
    }
}

/// An upper bound gives boundedness above.
theorem upper_bound_imp_bounded_above[L: LinearOrder](p: L -> Bool, b: L) {
    is_upper_bound(p, b) implies is_bounded_above(p)
} by {
    if is_upper_bound(p, b) {
        exists(x: L) {
            x = b and is_upper_bound(p, x)
        }
        is_bounded_above(p)
    }
}

/// Boundedness by an interval gives boundedness below.
theorem bounded_by_interval_imp_bounded_below[L: LinearOrder](p: L -> Bool, a: L, b: L) {
    is_bounded_by_interval(p, a, b) implies is_bounded_below(p)
} by {
    if is_bounded_by_interval(p, a, b) {
        bounded_by_interval_imp_lower_bound(p, a, b)
        is_lower_bound(p, a)
        lower_bound_imp_bounded_below(p, a)
        is_bounded_below(p)
    }
}

/// Boundedness by an interval gives boundedness above.
theorem bounded_by_interval_imp_bounded_above[L: LinearOrder](p: L -> Bool, a: L, b: L) {
    is_bounded_by_interval(p, a, b) implies is_bounded_above(p)
} by {
    if is_bounded_by_interval(p, a, b) {
        bounded_by_interval_imp_upper_bound(p, a, b)
        is_upper_bound(p, b)
        upper_bound_imp_bounded_above(p, b)
        is_bounded_above(p)
    }
}

/// An interval bound gives boundedness.
theorem bounded_by_interval_imp_bounded[L: LinearOrder](p: L -> Bool, a: L, b: L) {
    is_bounded_by_interval(p, a, b) implies is_bounded(p)
} by {
    if is_bounded_by_interval(p, a, b) {
        exists(x: L, y: L) {
            x = a and y = b and is_bounded_by_interval(p, x, y)
        }
        is_bounded(p)
    }
}

/// Boundedness gives boundedness below.
theorem bounded_imp_bounded_below[L: LinearOrder](p: L -> Bool) {
    is_bounded(p) implies is_bounded_below(p)
} by {
    if is_bounded(p) {
        let (a: L, b: L) satisfy {
            is_bounded_by_interval(p, a, b)
        }
        bounded_by_interval_imp_bounded_below(p, a, b)
        is_bounded_below(p)
    }
}

/// Boundedness gives boundedness above.
theorem bounded_imp_bounded_above[L: LinearOrder](p: L -> Bool) {
    is_bounded(p) implies is_bounded_above(p)
} by {
    if is_bounded(p) {
        let (a: L, b: L) satisfy {
            is_bounded_by_interval(p, a, b)
        }
        bounded_by_interval_imp_bounded_above(p, a, b)
        is_bounded_above(p)
    }
}

/// Lower and upper bounds give boundedness.
theorem bounds_imp_bounded[L: LinearOrder](p: L -> Bool, a: L, b: L) {
    is_lower_bound(p, a) and is_upper_bound(p, b) implies is_bounded(p)
} by {
    if is_lower_bound(p, a) and is_upper_bound(p, b) {
        bounds_imp_bounded_by_interval(p, a, b)
        is_bounded_by_interval(p, a, b)
        bounded_by_interval_imp_bounded(p, a, b)
        is_bounded(p)
    }
}
