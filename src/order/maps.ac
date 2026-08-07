/// Order-preserving and order-reversing maps.

from data.basic.functions import compose, identity_fn, is_injective_fn, is_surjective_fn, compose_surjective_fn
from order.base import PartialOrder, LinearOrder, lte_refl, lte_antisymm,
    lt_imp_lte, not_lte_imp_gt, gt_imp_not_lte,
    lte_min_of_bounds, max_lte_of_upper_bounds,
    min_lte_left, min_lte_right, lte_max_left, lte_max_right, min_eq_left_of_lte,
    min_eq_right_of_gte, max_eq_left_of_gte, max_eq_right_of_lte
from order.interval import closed_interval, open_interval, left_open_interval, right_open_interval,
    clamp

/// True if a map preserves the non-strict order.
define is_monotone[A: PartialOrder, B: PartialOrder](f: A -> B) -> Bool {
    forall(x: A, y: A) {
        x <= y implies f(x) <= f(y)
    }
}

/// True if a map reverses the non-strict order.
define is_antitone[A: PartialOrder, B: PartialOrder](f: A -> B) -> Bool {
    forall(x: A, y: A) {
        x <= y implies f(y) <= f(x)
    }
}

/// True if a map preserves strict order.
define is_strict_monotone[A: PartialOrder, B: PartialOrder](f: A -> B) -> Bool {
    forall(x: A, y: A) {
        x < y implies f(x) < f(y)
    }
}

/// True if a map reverses strict order.
define is_strict_antitone[A: PartialOrder, B: PartialOrder](f: A -> B) -> Bool {
    forall(x: A, y: A) {
        x < y implies f(y) < f(x)
    }
}

/// True if a map reflects and preserves the non-strict order.
define is_order_embedding[A: PartialOrder, B: PartialOrder](f: A -> B) -> Bool {
    forall(x: A, y: A) {
        f(x) <= f(y) = (x <= y)
    }
}

/// True if a map is monotone and surjective.
define is_order_surjection[A: PartialOrder, B: PartialOrder](f: A -> B) -> Bool {
    is_monotone(f) and is_surjective_fn(f)
}

/// A monotone map carries an explicitly ordered pair to an ordered pair.
theorem monotone_step[A: PartialOrder, B: PartialOrder](f: A -> B, x: A, y: A) {
    is_monotone(f) and x <= y implies f(x) <= f(y)
} by {
    if is_monotone(f) and x <= y {
        is_monotone(f) = forall(a: A, b: A) {
            a <= b implies f(a) <= f(b)
        }
        f(x) <= f(y)
    }
}

/// A monotone map carries an ordered pair to an ordered pair.
theorem monotone_apply[A: PartialOrder, B: PartialOrder](f: A -> B, x: A, y: A) {
    is_monotone(f) and x <= y implies f(x) <= f(y)
} by {
    if is_monotone(f) and x <= y {
        monotone_step(f, x, y)
        f(x) <= f(y)
    }
}

/// A map is monotone when it carries each ordered pair to an ordered pair.
theorem monotone_from_forall[A: PartialOrder, B: PartialOrder](f: A -> B) {
    forall(x: A, y: A) { x <= y implies f(x) <= f(y) } implies is_monotone(f)
} by {
    if forall(x: A, y: A) { x <= y implies f(x) <= f(y) } {
        is_monotone(f) = forall(x: A, y: A) {
            x <= y implies f(x) <= f(y)
        }
        is_monotone(f)
    }
}

/// An antitone map reverses an explicitly ordered pair.
theorem antitone_step[A: PartialOrder, B: PartialOrder](f: A -> B, x: A, y: A) {
    is_antitone(f) and x <= y implies f(y) <= f(x)
} by {
    if is_antitone(f) and x <= y {
        is_antitone(f) = forall(a: A, b: A) {
            a <= b implies f(b) <= f(a)
        }
        f(y) <= f(x)
    }
}

/// An antitone map reverses an ordered pair.
theorem antitone_apply[A: PartialOrder, B: PartialOrder](f: A -> B, x: A, y: A) {
    is_antitone(f) and x <= y implies f(y) <= f(x)
} by {
    if is_antitone(f) and x <= y {
        antitone_step(f, x, y)
        f(y) <= f(x)
    }
}

/// A map is antitone when it reverses each ordered pair.
theorem antitone_from_forall[A: PartialOrder, B: PartialOrder](f: A -> B) {
    forall(x: A, y: A) { x <= y implies f(y) <= f(x) } implies is_antitone(f)
} by {
    if forall(x: A, y: A) { x <= y implies f(y) <= f(x) } {
        is_antitone(f) = forall(x: A, y: A) {
            x <= y implies f(y) <= f(x)
        }
        is_antitone(f)
    }
}

/// A strict monotone map carries an explicitly strictly ordered pair to a strictly ordered pair.
theorem strict_monotone_step[A: PartialOrder, B: PartialOrder](f: A -> B, x: A, y: A) {
    is_strict_monotone(f) and x < y implies f(x) < f(y)
} by {
    if is_strict_monotone(f) and x < y {
        is_strict_monotone(f) = forall(a: A, b: A) {
            a < b implies f(a) < f(b)
        }
        f(x) < f(y)
    }
}

/// A strict monotone map carries a strictly ordered pair to a strictly ordered pair.
theorem strict_monotone_apply[A: PartialOrder, B: PartialOrder](f: A -> B, x: A, y: A) {
    is_strict_monotone(f) and x < y implies f(x) < f(y)
} by {
    if is_strict_monotone(f) and x < y {
        strict_monotone_step(f, x, y)
        f(x) < f(y)
    }
}

/// A strict antitone map reverses an explicitly strictly ordered pair.
theorem strict_antitone_step[A: PartialOrder, B: PartialOrder](f: A -> B, x: A, y: A) {
    is_strict_antitone(f) and x < y implies f(y) < f(x)
} by {
    if is_strict_antitone(f) and x < y {
        is_strict_antitone(f) = forall(a: A, b: A) {
            a < b implies f(b) < f(a)
        }
        f(y) < f(x)
    }
}

/// A strict antitone map reverses a strictly ordered pair.
theorem strict_antitone_apply[A: PartialOrder, B: PartialOrder](f: A -> B, x: A, y: A) {
    is_strict_antitone(f) and x < y implies f(y) < f(x)
} by {
    if is_strict_antitone(f) and x < y {
        strict_antitone_step(f, x, y)
        f(y) < f(x)
    }
}

/// An order embedding preserves an explicitly ordered pair.
theorem order_embedding_monotone_step[A: PartialOrder, B: PartialOrder](f: A -> B, x: A, y: A) {
    is_order_embedding(f) and x <= y implies f(x) <= f(y)
} by {
    if is_order_embedding(f) and x <= y {
        is_order_embedding(f) = forall(a: A, b: A) {
            f(a) <= f(b) = (a <= b)
        }
        f(x) <= f(y)
    }
}

/// An order embedding preserves an ordered pair.
theorem order_embedding_apply_le[A: PartialOrder, B: PartialOrder](f: A -> B, x: A, y: A) {
    is_order_embedding(f) and x <= y implies f(x) <= f(y)
} by {
    if is_order_embedding(f) and x <= y {
        order_embedding_monotone_step(f, x, y)
        f(x) <= f(y)
    }
}

/// An order embedding reflects an explicitly ordered image pair.
theorem order_embedding_reflects_lte[A: PartialOrder, B: PartialOrder](f: A -> B, x: A, y: A) {
    is_order_embedding(f) and f(x) <= f(y) implies x <= y
} by {
    if is_order_embedding(f) and f(x) <= f(y) {
        is_order_embedding(f) = forall(a: A, b: A) {
            f(a) <= f(b) = (a <= b)
        }
        x <= y
    }
}

/// An order embedding reflects an ordered image pair.
theorem order_embedding_reflects_le[A: PartialOrder, B: PartialOrder](f: A -> B, x: A, y: A) {
    is_order_embedding(f) and f(x) <= f(y) implies x <= y
} by {
    if is_order_embedding(f) and f(x) <= f(y) {
        order_embedding_reflects_lte(f, x, y)
        x <= y
    }
}

/// An order embedding preserves and reflects non-strict order.
theorem order_embedding_le_iff_le[A: PartialOrder, B: PartialOrder](f: A -> B, x: A, y: A) {
    is_order_embedding(f) implies (f(x) <= f(y) = (x <= y))
} by {
    if is_order_embedding(f) {
        if f(x) <= f(y) {
            order_embedding_reflects_lte(f, x, y)
            x <= y
        }
        if x <= y {
            order_embedding_monotone_step(f, x, y)
            f(x) <= f(y)
        }
        f(x) <= f(y) = (x <= y)
    }
}

/// An order embedding reflects an explicitly reversed ordered image pair.
theorem order_embedding_reflects_gte[A: PartialOrder, B: PartialOrder](f: A -> B, x: A, y: A) {
    is_order_embedding(f) and f(x) >= f(y) implies x >= y
} by {
    if is_order_embedding(f) and f(x) >= f(y) {
        f(y) <= f(x)
        order_embedding_reflects_lte(f, y, x)
        y <= x
        x >= y
    }
}

/// An order embedding reflects a reverse ordered image pair.
theorem order_embedding_reflects_ge[A: PartialOrder, B: PartialOrder](f: A -> B, x: A, y: A) {
    is_order_embedding(f) and f(x) >= f(y) implies x >= y
} by {
    if is_order_embedding(f) and f(x) >= f(y) {
        order_embedding_reflects_gte(f, x, y)
        x >= y
    }
}

/// An order embedding preserves an explicitly strict ordered pair.
theorem order_embedding_strict_monotone_step[A: PartialOrder, B: PartialOrder](f: A -> B, x: A, y: A) {
    is_order_embedding(f) and x < y implies f(x) < f(y)
} by {
    if is_order_embedding(f) and x < y {
        x <= y
        order_embedding_monotone_step(f, x, y)
        f(x) <= f(y)
        if f(x) = f(y) {
            f(y) <= f(x)
            order_embedding_reflects_lte(f, y, x)
            y <= x
            lte_antisymm(x, y)
            x = y
            x != y
            false
        }
        f(x) != f(y)
        f(x) < f(y)
    }
}

/// An order embedding preserves a strictly ordered pair.
theorem order_embedding_apply_lt[A: PartialOrder, B: PartialOrder](f: A -> B, x: A, y: A) {
    is_order_embedding(f) and x < y implies f(x) < f(y)
} by {
    if is_order_embedding(f) and x < y {
        order_embedding_strict_monotone_step(f, x, y)
        f(x) < f(y)
    }
}

/// An order embedding preserves a reverse ordered pair.
theorem order_embedding_apply_ge[A: PartialOrder, B: PartialOrder](f: A -> B, x: A, y: A) {
    is_order_embedding(f) and x >= y implies f(x) >= f(y)
} by {
    if is_order_embedding(f) and x >= y {
        y <= x
        order_embedding_monotone_step(f, y, x)
        f(y) <= f(x)
        f(x) >= f(y)
    }
}

/// An order embedding preserves and reflects reverse non-strict order.
theorem order_embedding_ge_iff_ge[A: PartialOrder, B: PartialOrder](f: A -> B, x: A, y: A) {
    is_order_embedding(f) implies (f(x) >= f(y) = (x >= y))
} by {
    if is_order_embedding(f) {
        if f(x) >= f(y) {
            order_embedding_reflects_gte(f, x, y)
            x >= y
        }
        if x >= y {
            order_embedding_apply_ge(f, x, y)
            f(x) >= f(y)
        }
        f(x) >= f(y) = (x >= y)
    }
}

/// An order embedding preserves an explicitly reversed strict ordered pair.
theorem order_embedding_strict_antitone_step_swap[A: PartialOrder, B: PartialOrder](f: A -> B, x: A, y: A) {
    is_order_embedding(f) and x > y implies f(x) > f(y)
} by {
    if is_order_embedding(f) and x > y {
        y < x
        order_embedding_strict_monotone_step(f, y, x)
        f(y) < f(x)
        f(x) > f(y)
    }
}

/// An order embedding preserves a reverse strictly ordered pair.
theorem order_embedding_apply_gt[A: PartialOrder, B: PartialOrder](f: A -> B, x: A, y: A) {
    is_order_embedding(f) and x > y implies f(x) > f(y)
} by {
    if is_order_embedding(f) and x > y {
        order_embedding_strict_antitone_step_swap(f, x, y)
        f(x) > f(y)
    }
}

/// An order embedding reflects an explicitly strict image pair.
theorem order_embedding_reflects_lt[A: PartialOrder, B: PartialOrder](f: A -> B, x: A, y: A) {
    is_order_embedding(f) and f(x) < f(y) implies x < y
} by {
    if is_order_embedding(f) and f(x) < f(y) {
        f(x) <= f(y)
        order_embedding_reflects_lte(f, x, y)
        x <= y
        f(x) != f(y)
        if x = y {
            f(x) = f(y)
            false
        }
        x != y
        x < y
    }
}

/// An order embedding preserves and reflects strict order.
theorem order_embedding_lt_iff_lt[A: PartialOrder, B: PartialOrder](f: A -> B, x: A, y: A) {
    is_order_embedding(f) implies (f(x) < f(y) = (x < y))
} by {
    if is_order_embedding(f) {
        if f(x) < f(y) {
            order_embedding_reflects_lt(f, x, y)
            x < y
        }
        if x < y {
            order_embedding_strict_monotone_step(f, x, y)
            f(x) < f(y)
        }
        f(x) < f(y) = (x < y)
    }
}

/// An order embedding reflects an explicitly reversed strict image pair.
theorem order_embedding_reflects_gt[A: PartialOrder, B: PartialOrder](f: A -> B, x: A, y: A) {
    is_order_embedding(f) and f(x) > f(y) implies x > y
} by {
    if is_order_embedding(f) and f(x) > f(y) {
        f(y) < f(x)
        order_embedding_reflects_lt(f, y, x)
        y < x
        x > y
    }
}

/// An order embedding preserves and reflects reverse strict order.
theorem order_embedding_gt_iff_gt[A: PartialOrder, B: PartialOrder](f: A -> B, x: A, y: A) {
    is_order_embedding(f) implies (f(x) > f(y) = (x > y))
} by {
    if is_order_embedding(f) {
        if f(x) > f(y) {
            order_embedding_reflects_gt(f, x, y)
            x > y
        }
        if x > y {
            order_embedding_strict_antitone_step_swap(f, x, y)
            f(x) > f(y)
        }
        f(x) > f(y) = (x > y)
    }
}

/// An order embedding is monotone.
theorem order_embedding_is_monotone[A: PartialOrder, B: PartialOrder](f: A -> B) {
    is_order_embedding(f) implies is_monotone(f)
} by {
    if is_order_embedding(f) {
        forall(x: A, y: A) {
            if x <= y {
                order_embedding_monotone_step(f, x, y)
                f(x) <= f(y)
            }
        }
    }
}

/// An order embedding is strict monotone.
theorem order_embedding_is_strict_monotone[A: PartialOrder, B: PartialOrder](f: A -> B) {
    is_order_embedding(f) implies is_strict_monotone(f)
} by {
    if is_order_embedding(f) {
        forall(x: A, y: A) {
            if x < y {
                order_embedding_strict_monotone_step(f, x, y)
                f(x) < f(y)
            }
        }
    }
}

/// An order-preserving surjection is monotone.
theorem order_surjection_is_monotone[A: PartialOrder, B: PartialOrder](f: A -> B) {
    is_order_surjection(f) implies is_monotone(f)
} by {
    if is_order_surjection(f) {
        is_order_surjection(f) = (is_monotone(f) and is_surjective_fn(f))
        is_monotone(f)
    }
}

/// An order-preserving surjection is surjective.
theorem order_surjection_is_surjective[A: PartialOrder, B: PartialOrder](f: A -> B) {
    is_order_surjection(f) implies is_surjective_fn(f)
} by {
    if is_order_surjection(f) {
        is_order_surjection(f) = (is_monotone(f) and is_surjective_fn(f))
        is_surjective_fn(f)
    }
}

/// An order embedding is injective.
theorem order_embedding_is_injective[A: PartialOrder, B: PartialOrder](f: A -> B) {
    is_order_embedding(f) implies forall(x: A, y: A) {
        f(x) = f(y) implies x = y
    }
} by {
    if is_order_embedding(f) {
        forall(x: A, y: A) {
            if f(x) = f(y) {
                lte_refl(f(x))
                f(x) <= f(y)
                order_embedding_reflects_lte(f, x, y)
                x <= y
                f(y) <= f(x)
                order_embedding_reflects_lte(f, y, x)
                y <= x
                lte_antisymm(x, y)
                x = y
            }
        }
    }
}

/// An order embedding is injective as a function.
theorem order_embedding_is_injective_fn[A: PartialOrder, B: PartialOrder](f: A -> B) {
    is_order_embedding(f) implies is_injective_fn(f)
} by {
    if is_order_embedding(f) {
        forall(x: A, y: A) {
            if f(x) = f(y) {
                order_embedding_is_injective(f)
                x = y
            }
        }
    }
}

/// A strict monotone map between linear orders preserves non-strict order.
theorem strict_monotone_preserves_le_of_linear_order[A: LinearOrder, B: LinearOrder](
    f: A -> B, x: A, y: A
) {
    is_strict_monotone(f) and x <= y implies f(x) <= f(y)
} by {
    if is_strict_monotone(f) and x <= y {
        if x = y {
            f(x) = f(y)
            lte_refl(f(y))
            f(x) <= f(y)
        } else {
            x != y
            x < y
            strict_monotone_step(f, x, y)
            f(x) < f(y)
            lt_imp_lte(f(x), f(y))
            f(x) <= f(y)
        }
    }
}

/// A strict monotone map between linear orders reflects non-strict order.
theorem strict_monotone_reflects_le_of_linear_order[A: LinearOrder, B: LinearOrder](
    f: A -> B, x: A, y: A
) {
    is_strict_monotone(f) and f(x) <= f(y) implies x <= y
} by {
    if is_strict_monotone(f) and f(x) <= f(y) {
        if not x <= y {
            not_lte_imp_gt(x, y)
            x > y
            y < x
            strict_monotone_step(f, y, x)
            f(y) < f(x)
            f(x) > f(y)
            gt_imp_not_lte(f(x), f(y))
            not f(x) <= f(y)
            false
        }
    }
}

/// A strict monotone map between linear orders is an order embedding.
theorem strict_monotone_is_order_embedding[A: LinearOrder, B: LinearOrder](f: A -> B) {
    is_strict_monotone(f) implies is_order_embedding(f)
} by {
    if is_strict_monotone(f) {
        forall(x: A, y: A) {
            if f(x) <= f(y) {
                strict_monotone_reflects_le_of_linear_order(f, x, y)
                x <= y
            }
            if x <= y {
                strict_monotone_preserves_le_of_linear_order(f, x, y)
                f(x) <= f(y)
            }
            f(x) <= f(y) = (x <= y)
        }
    }
}

/// A strict monotone map between linear orders is injective as a function.
theorem strict_monotone_is_injective_fn[A: LinearOrder, B: LinearOrder](f: A -> B) {
    is_strict_monotone(f) implies is_injective_fn(f)
} by {
    if is_strict_monotone(f) {
        strict_monotone_is_order_embedding(f)
        is_order_embedding(f)
        order_embedding_is_injective_fn(f)
        is_injective_fn(f)
    }
}

/// The identity map is monotone.
theorem identity_is_monotone[A: PartialOrder] {
    is_monotone(identity_fn[A])
} by {
    forall(x: A, y: A) {
        if x <= y {
            identity_fn[A](x) = x
            identity_fn[A](y) = y
            identity_fn[A](x) <= identity_fn[A](y)
        }
    }
}

/// The identity map is monotone.
theorem monotone_id[A: PartialOrder] {
    is_monotone(identity_fn[A])
} by {
    identity_is_monotone[A]
}

/// The identity map is antitone only on explicitly comparable equal pairs.
theorem identity_antitone_step_of_eq[A: PartialOrder](x: A, y: A) {
    x = y implies identity_fn[A](y) <= identity_fn[A](x)
} by {
    if x = y {
        identity_fn[A](y) = y
        identity_fn[A](x) = x
        lte_refl(x)
        identity_fn[A](y) <= identity_fn[A](x)
    }
}

/// The identity map is strict monotone.
theorem identity_is_strict_monotone[A: PartialOrder] {
    is_strict_monotone(identity_fn[A])
} by {
    forall(x: A, y: A) {
        if x < y {
            identity_fn[A](x) = x
            identity_fn[A](y) = y
            identity_fn[A](x) < identity_fn[A](y)
        }
    }
}

/// The identity map is strict monotone.
theorem strict_monotone_id[A: PartialOrder] {
    is_strict_monotone(identity_fn[A])
} by {
    identity_is_strict_monotone[A]
}

/// The identity map is an order embedding.
theorem identity_is_order_embedding[A: PartialOrder] {
    is_order_embedding(identity_fn[A])
} by {
    forall(x: A, y: A) {
        identity_fn[A](x) = x
        identity_fn[A](y) = y
        identity_fn[A](x) <= identity_fn[A](y) = (x <= y)
    }
}

/// The identity map is an order embedding.
theorem order_embedding_id[A: PartialOrder] {
    is_order_embedding(identity_fn[A])
} by {
    identity_is_order_embedding[A]
}

/// The identity map is injective as a function.
theorem identity_is_injective_fn[A: PartialOrder] {
    is_injective_fn(identity_fn[A])
} by {
    identity_is_order_embedding[A]
    order_embedding_is_injective_fn(identity_fn[A])
}

/// The identity map is surjective.
theorem identity_is_surjective_fn[A: PartialOrder] {
    is_surjective_fn(identity_fn[A])
} by {
    forall(y: A) {
        exists(x: A) {
            x = y and identity_fn[A](x) = y
        }
    }
}

/// The identity map is an order-preserving surjection.
theorem identity_is_order_surjection[A: PartialOrder] {
    is_order_surjection(identity_fn[A])
} by {
    identity_is_monotone[A]
    is_monotone(identity_fn[A])
    identity_is_surjective_fn[A]
    is_surjective_fn(identity_fn[A])
    is_order_surjection(identity_fn[A])
}

/// The composite of monotone maps is monotone.
theorem monotone_compose[A: PartialOrder, B: PartialOrder, C: PartialOrder](g: A -> B, f: B -> C) {
    is_monotone(g) and is_monotone(f) implies is_monotone(compose(f, g))
} by {
    if is_monotone(g) and is_monotone(f) {
        forall(x: A, y: A) {
            if x <= y {
                monotone_step(g, x, y)
                g(x) <= g(y)
                monotone_step(f, g(x), g(y))
                f(g(x)) <= f(g(y))
                compose(f, g, x) = f(g(x))
                compose(f, g, y) = f(g(y))
                compose(f, g)(x) <= compose(f, g)(y)
            }
        }
    }
}

/// The composite of monotone maps is monotone.
theorem monotone_comp[A: PartialOrder, B: PartialOrder, C: PartialOrder](g: A -> B, f: B -> C) {
    is_monotone(g) and is_monotone(f) implies is_monotone(compose(f, g))
} by {
    if is_monotone(g) and is_monotone(f) {
        monotone_compose(g, f)
        is_monotone(compose(f, g))
    }
}

/// The composite of order-preserving surjections is an order-preserving surjection.
theorem order_surjection_compose[A: PartialOrder, B: PartialOrder, C: PartialOrder](g: A -> B, f: B -> C) {
    is_order_surjection(g) and is_order_surjection(f) implies is_order_surjection(compose(f, g))
} by {
    if is_order_surjection(g) and is_order_surjection(f) {
        order_surjection_is_monotone(g)
        is_monotone(g)
        order_surjection_is_monotone(f)
        is_monotone(f)
        monotone_compose(g, f)
        is_monotone(compose(f, g))
        order_surjection_is_surjective(g)
        is_surjective_fn(g)
        order_surjection_is_surjective(f)
        is_surjective_fn(f)
        compose_surjective_fn(f, g)
        is_surjective_fn(compose(f, g))
        is_order_surjection(compose(f, g))
    }
}

/// The composite of order-preserving surjections is an order-preserving surjection.
theorem order_surjection_comp[A: PartialOrder, B: PartialOrder, C: PartialOrder](g: A -> B, f: B -> C) {
    is_order_surjection(g) and is_order_surjection(f) implies is_order_surjection(compose(f, g))
} by {
    if is_order_surjection(g) and is_order_surjection(f) {
        order_surjection_compose(g, f)
        is_order_surjection(compose(f, g))
    }
}

/// The composite of two antitone maps is monotone.
theorem antitone_compose_antitone[A: PartialOrder, B: PartialOrder, C: PartialOrder](g: A -> B, f: B -> C) {
    is_antitone(g) and is_antitone(f) implies is_monotone(compose(f, g))
} by {
    if is_antitone(g) and is_antitone(f) {
        forall(x: A, y: A) {
            if x <= y {
                antitone_step(g, x, y)
                g(y) <= g(x)
                antitone_step(f, g(y), g(x))
                f(g(x)) <= f(g(y))
                compose(f, g, x) = f(g(x))
                compose(f, g, y) = f(g(y))
                compose(f, g)(x) <= compose(f, g)(y)
            }
        }
    }
}

/// The composite of two antitone maps is monotone.
theorem antitone_comp_antitone[A: PartialOrder, B: PartialOrder, C: PartialOrder](g: A -> B, f: B -> C) {
    is_antitone(g) and is_antitone(f) implies is_monotone(compose(f, g))
} by {
    if is_antitone(g) and is_antitone(f) {
        antitone_compose_antitone(g, f)
        is_monotone(compose(f, g))
    }
}

/// A monotone map followed by an antitone map is antitone.
theorem monotone_compose_antitone[A: PartialOrder, B: PartialOrder, C: PartialOrder](g: A -> B, f: B -> C) {
    is_monotone(g) and is_antitone(f) implies is_antitone(compose(f, g))
} by {
    if is_monotone(g) and is_antitone(f) {
        forall(x: A, y: A) {
            if x <= y {
                monotone_step(g, x, y)
                g(x) <= g(y)
                antitone_step(f, g(x), g(y))
                f(g(y)) <= f(g(x))
                compose(f, g, x) = f(g(x))
                compose(f, g, y) = f(g(y))
                compose(f, g)(y) <= compose(f, g)(x)
            }
        }
    }
}

/// A monotone map followed by an antitone map is antitone.
theorem monotone_comp_antitone[A: PartialOrder, B: PartialOrder, C: PartialOrder](g: A -> B, f: B -> C) {
    is_monotone(g) and is_antitone(f) implies is_antitone(compose(f, g))
} by {
    if is_monotone(g) and is_antitone(f) {
        monotone_compose_antitone(g, f)
        is_antitone(compose(f, g))
    }
}

/// An antitone map followed by a monotone map is antitone.
theorem antitone_compose_monotone[A: PartialOrder, B: PartialOrder, C: PartialOrder](g: A -> B, f: B -> C) {
    is_antitone(g) and is_monotone(f) implies is_antitone(compose(f, g))
} by {
    if is_antitone(g) and is_monotone(f) {
        forall(x: A, y: A) {
            if x <= y {
                antitone_step(g, x, y)
                g(y) <= g(x)
                monotone_step(f, g(y), g(x))
                f(g(y)) <= f(g(x))
                compose(f, g, x) = f(g(x))
                compose(f, g, y) = f(g(y))
                compose(f, g)(y) <= compose(f, g)(x)
            }
        }
    }
}

/// An antitone map followed by a monotone map is antitone.
theorem antitone_comp_monotone[A: PartialOrder, B: PartialOrder, C: PartialOrder](g: A -> B, f: B -> C) {
    is_antitone(g) and is_monotone(f) implies is_antitone(compose(f, g))
} by {
    if is_antitone(g) and is_monotone(f) {
        antitone_compose_monotone(g, f)
        is_antitone(compose(f, g))
    }
}

/// The composite of strict monotone maps is strict monotone.
theorem strict_monotone_compose[A: PartialOrder, B: PartialOrder, C: PartialOrder](g: A -> B, f: B -> C) {
    is_strict_monotone(g) and is_strict_monotone(f) implies is_strict_monotone(compose(f, g))
} by {
    if is_strict_monotone(g) and is_strict_monotone(f) {
        forall(x: A, y: A) {
            if x < y {
                strict_monotone_step(g, x, y)
                g(x) < g(y)
                strict_monotone_step(f, g(x), g(y))
                f(g(x)) < f(g(y))
                compose(f, g, x) = f(g(x))
                compose(f, g, y) = f(g(y))
                compose(f, g)(x) < compose(f, g)(y)
            }
        }
    }
}

/// The composite of strict monotone maps is strict monotone.
theorem strict_monotone_comp[A: PartialOrder, B: PartialOrder, C: PartialOrder](g: A -> B, f: B -> C) {
    is_strict_monotone(g) and is_strict_monotone(f) implies is_strict_monotone(compose(f, g))
} by {
    if is_strict_monotone(g) and is_strict_monotone(f) {
        strict_monotone_compose(g, f)
        is_strict_monotone(compose(f, g))
    }
}

/// The composite of strict antitone maps is strict monotone.
theorem strict_antitone_compose_strict_antitone[A: PartialOrder, B: PartialOrder, C: PartialOrder](g: A -> B, f: B -> C) {
    is_strict_antitone(g) and is_strict_antitone(f) implies is_strict_monotone(compose(f, g))
} by {
    if is_strict_antitone(g) and is_strict_antitone(f) {
        forall(x: A, y: A) {
            if x < y {
                strict_antitone_step(g, x, y)
                g(y) < g(x)
                strict_antitone_step(f, g(y), g(x))
                f(g(x)) < f(g(y))
                compose(f, g, x) = f(g(x))
                compose(f, g, y) = f(g(y))
                compose(f, g)(x) < compose(f, g)(y)
            }
        }
    }
}

/// The composite of strict antitone maps is strict monotone.
theorem strict_antitone_comp_strict_antitone[A: PartialOrder, B: PartialOrder, C: PartialOrder](g: A -> B, f: B -> C) {
    is_strict_antitone(g) and is_strict_antitone(f) implies is_strict_monotone(compose(f, g))
} by {
    if is_strict_antitone(g) and is_strict_antitone(f) {
        strict_antitone_compose_strict_antitone(g, f)
        is_strict_monotone(compose(f, g))
    }
}

/// A strict monotone map followed by a strict antitone map is strict antitone.
theorem strict_monotone_compose_strict_antitone[A: PartialOrder, B: PartialOrder, C: PartialOrder](g: A -> B, f: B -> C) {
    is_strict_monotone(g) and is_strict_antitone(f) implies is_strict_antitone(compose(f, g))
} by {
    if is_strict_monotone(g) and is_strict_antitone(f) {
        forall(x: A, y: A) {
            if x < y {
                strict_monotone_step(g, x, y)
                g(x) < g(y)
                strict_antitone_step(f, g(x), g(y))
                f(g(y)) < f(g(x))
                compose(f, g, x) = f(g(x))
                compose(f, g, y) = f(g(y))
                compose(f, g)(y) < compose(f, g)(x)
            }
        }
    }
}

/// A strict monotone map followed by a strict antitone map is strict antitone.
theorem strict_monotone_comp_strict_antitone[A: PartialOrder, B: PartialOrder, C: PartialOrder](g: A -> B, f: B -> C) {
    is_strict_monotone(g) and is_strict_antitone(f) implies is_strict_antitone(compose(f, g))
} by {
    if is_strict_monotone(g) and is_strict_antitone(f) {
        strict_monotone_compose_strict_antitone(g, f)
        is_strict_antitone(compose(f, g))
    }
}

/// A strict antitone map followed by a strict monotone map is strict antitone.
theorem strict_antitone_compose_strict_monotone[A: PartialOrder, B: PartialOrder, C: PartialOrder](g: A -> B, f: B -> C) {
    is_strict_antitone(g) and is_strict_monotone(f) implies is_strict_antitone(compose(f, g))
} by {
    if is_strict_antitone(g) and is_strict_monotone(f) {
        forall(x: A, y: A) {
            if x < y {
                strict_antitone_step(g, x, y)
                g(y) < g(x)
                strict_monotone_step(f, g(y), g(x))
                f(g(y)) < f(g(x))
                compose(f, g, x) = f(g(x))
                compose(f, g, y) = f(g(y))
                compose(f, g)(y) < compose(f, g)(x)
            }
        }
    }
}

/// A strict antitone map followed by a strict monotone map is strict antitone.
theorem strict_antitone_comp_strict_monotone[A: PartialOrder, B: PartialOrder, C: PartialOrder](g: A -> B, f: B -> C) {
    is_strict_antitone(g) and is_strict_monotone(f) implies is_strict_antitone(compose(f, g))
} by {
    if is_strict_antitone(g) and is_strict_monotone(f) {
        strict_antitone_compose_strict_monotone(g, f)
        is_strict_antitone(compose(f, g))
    }
}

/// The composite of order embeddings is an order embedding.
theorem order_embedding_compose[A: PartialOrder, B: PartialOrder, C: PartialOrder](g: A -> B, f: B -> C) {
    is_order_embedding(g) and is_order_embedding(f) implies is_order_embedding(compose(f, g))
} by {
    if is_order_embedding(g) and is_order_embedding(f) {
        forall(x: A, y: A) {
            compose(f, g, x) = f(g(x))
            compose(f, g, y) = f(g(y))
            if compose(f, g)(x) <= compose(f, g)(y) {
                f(g(x)) <= f(g(y))
                order_embedding_reflects_lte(f, g(x), g(y))
                g(x) <= g(y)
                order_embedding_reflects_lte(g, x, y)
                x <= y
            }
            if x <= y {
                order_embedding_monotone_step(g, x, y)
                g(x) <= g(y)
                order_embedding_monotone_step(f, g(x), g(y))
                f(g(x)) <= f(g(y))
                compose(f, g)(x) <= compose(f, g)(y)
            }
            compose(f, g)(x) <= compose(f, g)(y) = (x <= y)
        }
    }
}

/// The composite of order embeddings is an order embedding.
theorem order_embedding_comp[A: PartialOrder, B: PartialOrder, C: PartialOrder](g: A -> B, f: B -> C) {
    is_order_embedding(g) and is_order_embedding(f) implies is_order_embedding(compose(f, g))
} by {
    if is_order_embedding(g) and is_order_embedding(f) {
        order_embedding_compose(g, f)
        is_order_embedding(compose(f, g))
    }
}

/// A monotone map sends a minimum below the minimum of the images.
theorem monotone_min_image_lte[A: LinearOrder, B: LinearOrder](f: A -> B, a: A, b: A) {
    is_monotone(f) implies f(a.min(b)) <= f(a).min(f(b))
} by {
    if is_monotone(f) {
        min_lte_left(a, b)
        monotone_step(f, a.min(b), a)
        f(a.min(b)) <= f(a)
        min_lte_right(a, b)
        monotone_step(f, a.min(b), b)
        f(a.min(b)) <= f(b)
        lte_min_of_bounds(f(a.min(b)), f(a), f(b))
        f(a.min(b)) <= f(a).min(f(b))
    }
}

/// The maximum of the images of a monotone map is below the image of the maximum.
theorem monotone_max_image_lte[A: LinearOrder, B: LinearOrder](f: A -> B, a: A, b: A) {
    is_monotone(f) implies f(a).max(f(b)) <= f(a.max(b))
} by {
    if is_monotone(f) {
        lte_max_left(a, b)
        monotone_step(f, a, a.max(b))
        f(a) <= f(a.max(b))
        lte_max_right(a, b)
        monotone_step(f, b, a.max(b))
        f(b) <= f(a.max(b))
        max_lte_of_upper_bounds(f(a), f(b), f(a.max(b)))
        f(a).max(f(b)) <= f(a.max(b))
    }
}

/// An antitone map sends a maximum below the minimum of the images.
theorem antitone_max_image_lte_min[A: LinearOrder, B: LinearOrder](f: A -> B, a: A, b: A) {
    is_antitone(f) implies f(a.max(b)) <= f(a).min(f(b))
} by {
    if is_antitone(f) {
        lte_max_left(a, b)
        antitone_step(f, a, a.max(b))
        f(a.max(b)) <= f(a)
        lte_max_right(a, b)
        antitone_step(f, b, a.max(b))
        f(a.max(b)) <= f(b)
        lte_min_of_bounds(f(a.max(b)), f(a), f(b))
        f(a.max(b)) <= f(a).min(f(b))
    }
}

/// The maximum of the images of an antitone map is below the image of the minimum.
theorem antitone_min_image_lte_max[A: LinearOrder, B: LinearOrder](f: A -> B, a: A, b: A) {
    is_antitone(f) implies f(a).max(f(b)) <= f(a.min(b))
} by {
    if is_antitone(f) {
        min_lte_left(a, b)
        antitone_step(f, a.min(b), a)
        f(a) <= f(a.min(b))
        min_lte_right(a, b)
        antitone_step(f, a.min(b), b)
        f(b) <= f(a.min(b))
        max_lte_of_upper_bounds(f(a), f(b), f(a.min(b)))
        f(a).max(f(b)) <= f(a.min(b))
    }
}

/// A monotone map preserves binary minima.
theorem monotone_min_image_eq[A: LinearOrder, B: LinearOrder](f: A -> B, a: A, b: A) {
    is_monotone(f) implies f(a.min(b)) = f(a).min(f(b))
} by {
    if is_monotone(f) {
        if a <= b {
            min_eq_left_of_lte(a, b)
            a.min(b) = a
            monotone_step(f, a, b)
            f(a) <= f(b)
            min_eq_left_of_lte(f(a), f(b))
            f(a).min(f(b)) = f(a)
            f(a.min(b)) = f(a).min(f(b))
        } else {
            b <= a
            a >= b
            min_eq_right_of_gte(a, b)
            a.min(b) = b
            monotone_step(f, b, a)
            f(b) <= f(a)
            f(a) >= f(b)
            min_eq_right_of_gte(f(a), f(b))
            f(a).min(f(b)) = f(b)
            f(a.min(b)) = f(a).min(f(b))
        }
    }
}

/// A monotone map preserves binary maxima.
theorem monotone_max_image_eq[A: LinearOrder, B: LinearOrder](f: A -> B, a: A, b: A) {
    is_monotone(f) implies f(a.max(b)) = f(a).max(f(b))
} by {
    if is_monotone(f) {
        if a <= b {
            max_eq_right_of_lte(a, b)
            a.max(b) = b
            monotone_step(f, a, b)
            f(a) <= f(b)
            max_eq_right_of_lte(f(a), f(b))
            f(a).max(f(b)) = f(b)
            f(a.max(b)) = f(a).max(f(b))
        } else {
            b <= a
            a >= b
            max_eq_left_of_gte(a, b)
            a.max(b) = a
            monotone_step(f, b, a)
            f(b) <= f(a)
            f(a) >= f(b)
            max_eq_left_of_gte(f(a), f(b))
            f(a).max(f(b)) = f(a)
            f(a.max(b)) = f(a).max(f(b))
        }
    }
}

/// An antitone map sends binary minima to binary maxima.
theorem antitone_min_image_eq_max[A: LinearOrder, B: LinearOrder](f: A -> B, a: A, b: A) {
    is_antitone(f) implies f(a.min(b)) = f(a).max(f(b))
} by {
    if is_antitone(f) {
        if a <= b {
            min_eq_left_of_lte(a, b)
            a.min(b) = a
            antitone_step(f, a, b)
            f(b) <= f(a)
            f(a) >= f(b)
            max_eq_left_of_gte(f(a), f(b))
            f(a).max(f(b)) = f(a)
            f(a.min(b)) = f(a).max(f(b))
        } else {
            b <= a
            a >= b
            min_eq_right_of_gte(a, b)
            a.min(b) = b
            antitone_step(f, b, a)
            f(a) <= f(b)
            max_eq_right_of_lte(f(a), f(b))
            f(a).max(f(b)) = f(b)
            f(a.min(b)) = f(a).max(f(b))
        }
    }
}

/// An antitone map sends binary maxima to binary minima.
theorem antitone_max_image_eq_min[A: LinearOrder, B: LinearOrder](f: A -> B, a: A, b: A) {
    is_antitone(f) implies f(a.max(b)) = f(a).min(f(b))
} by {
    if is_antitone(f) {
        if a <= b {
            max_eq_right_of_lte(a, b)
            a.max(b) = b
            antitone_step(f, a, b)
            f(b) <= f(a)
            f(a) >= f(b)
            min_eq_right_of_gte(f(a), f(b))
            f(a).min(f(b)) = f(b)
            f(a.max(b)) = f(a).min(f(b))
        } else {
            b <= a
            a >= b
            max_eq_left_of_gte(a, b)
            a.max(b) = a
            antitone_step(f, b, a)
            f(a) <= f(b)
            min_eq_left_of_lte(f(a), f(b))
            f(a).min(f(b)) = f(a)
            f(a.max(b)) = f(a).min(f(b))
        }
    }
}

/// The minimum of the images of a monotone map is below the image of the minimum.
theorem monotone_min_images_lte_image_min[A: LinearOrder, B: LinearOrder](f: A -> B, a: A, b: A) {
    is_monotone(f) implies f(a).min(f(b)) <= f(a.min(b))
} by {
    if is_monotone(f) {
        monotone_min_image_eq(f, a, b)
        f(a.min(b)) = f(a).min(f(b))
        lte_refl(f(a).min(f(b)))
        f(a).min(f(b)) <= f(a.min(b))
    }
}

/// The image of the maximum of a monotone map is below the maximum of the images.
theorem monotone_image_max_lte_max_images[A: LinearOrder, B: LinearOrder](f: A -> B, a: A, b: A) {
    is_monotone(f) implies f(a.max(b)) <= f(a).max(f(b))
} by {
    if is_monotone(f) {
        monotone_max_image_eq(f, a, b)
        f(a.max(b)) = f(a).max(f(b))
        lte_refl(f(a).max(f(b)))
        f(a.max(b)) <= f(a).max(f(b))
    }
}

/// The maximum of the images of an antitone map is below the image of the minimum.
theorem antitone_max_images_lte_image_min[A: LinearOrder, B: LinearOrder](f: A -> B, a: A, b: A) {
    is_antitone(f) implies f(a).max(f(b)) <= f(a.min(b))
} by {
    if is_antitone(f) {
        antitone_min_image_eq_max(f, a, b)
        f(a.min(b)) = f(a).max(f(b))
        lte_refl(f(a).max(f(b)))
        f(a).max(f(b)) <= f(a.min(b))
    }
}

/// The image of the maximum of an antitone map is below the minimum of the images.
theorem antitone_image_max_lte_min_images[A: LinearOrder, B: LinearOrder](f: A -> B, a: A, b: A) {
    is_antitone(f) implies f(a.max(b)) <= f(a).min(f(b))
} by {
    if is_antitone(f) {
        antitone_max_image_eq_min(f, a, b)
        f(a.max(b)) = f(a).min(f(b))
        lte_refl(f(a).min(f(b)))
        f(a.max(b)) <= f(a).min(f(b))
    }
}

/// A monotone map preserves binary minima.
theorem monotone_preserves_min[A: LinearOrder, B: LinearOrder](f: A -> B, a: A, b: A) {
    is_monotone(f) implies f(a.min(b)) = f(a).min(f(b))
} by {
    monotone_min_image_eq(f, a, b)
    f(a.min(b)) = f(a).min(f(b))
}

/// A monotone map preserves binary maxima.
theorem monotone_preserves_max[A: LinearOrder, B: LinearOrder](f: A -> B, a: A, b: A) {
    is_monotone(f) implies f(a.max(b)) = f(a).max(f(b))
} by {
    monotone_max_image_eq(f, a, b)
    f(a.max(b)) = f(a).max(f(b))
}

/// An antitone map exchanges binary minima with binary maxima.
theorem antitone_sends_min_to_max[A: LinearOrder, B: LinearOrder](f: A -> B, a: A, b: A) {
    is_antitone(f) implies f(a.min(b)) = f(a).max(f(b))
} by {
    antitone_min_image_eq_max(f, a, b)
    f(a.min(b)) = f(a).max(f(b))
}

/// An antitone map exchanges binary maxima with binary minima.
theorem antitone_sends_max_to_min[A: LinearOrder, B: LinearOrder](f: A -> B, a: A, b: A) {
    is_antitone(f) implies f(a.max(b)) = f(a).min(f(b))
} by {
    antitone_max_image_eq_min(f, a, b)
    f(a.max(b)) = f(a).min(f(b))
}

/// An order embedding preserves binary minima.
theorem order_embedding_min_image[A: LinearOrder, B: LinearOrder](f: A -> B, a: A, b: A) {
    is_order_embedding(f) implies f(a.min(b)) = f(a).min(f(b))
} by {
    if is_order_embedding(f) {
        order_embedding_is_monotone(f)
        is_monotone(f)
        if a <= b {
            min_eq_left_of_lte(a, b)
            a.min(b) = a
            order_embedding_monotone_step(f, a, b)
            f(a) <= f(b)
            min_eq_left_of_lte(f(a), f(b))
            f(a).min(f(b)) = f(a)
            f(a.min(b)) = f(a).min(f(b))
        } else {
            b <= a
            a >= b
            min_eq_right_of_gte(a, b)
            a.min(b) = b
            order_embedding_monotone_step(f, b, a)
            f(b) <= f(a)
            f(a) >= f(b)
            min_eq_right_of_gte(f(a), f(b))
            f(a).min(f(b)) = f(b)
            f(a.min(b)) = f(a).min(f(b))
        }
    }
}

/// An order embedding preserves binary maxima.
theorem order_embedding_max_image[A: LinearOrder, B: LinearOrder](f: A -> B, a: A, b: A) {
    is_order_embedding(f) implies f(a.max(b)) = f(a).max(f(b))
} by {
    if is_order_embedding(f) {
        order_embedding_is_monotone(f)
        is_monotone(f)
        if a <= b {
            max_eq_right_of_lte(a, b)
            a.max(b) = b
            order_embedding_monotone_step(f, a, b)
            f(a) <= f(b)
            max_eq_right_of_lte(f(a), f(b))
            f(a).max(f(b)) = f(b)
            f(a.max(b)) = f(a).max(f(b))
        } else {
            b <= a
            a >= b
            max_eq_left_of_gte(a, b)
            a.max(b) = a
            order_embedding_monotone_step(f, b, a)
            f(b) <= f(a)
            f(a) >= f(b)
            max_eq_left_of_gte(f(a), f(b))
            f(a).max(f(b)) = f(a)
            f(a.max(b)) = f(a).max(f(b))
        }
    }
}

/// A monotone map sends closed intervals into closed intervals between the image endpoints.
theorem monotone_closed_interval_image[A: LinearOrder, B: LinearOrder](f: A -> B, a: A, b: A, x: A) {
    is_monotone(f) and closed_interval(a, b, x) implies closed_interval(f(a), f(b), f(x))
} by {
    if is_monotone(f) and closed_interval(a, b, x) {
        a <= x
        monotone_step(f, a, x)
        f(a) <= f(x)
        x <= b
        monotone_step(f, x, b)
        f(x) <= f(b)
        closed_interval(f(a), f(b), f(x))
    }
}

/// A strict monotone map sends open intervals into open intervals between the image endpoints.
theorem strict_monotone_open_interval_image[A: LinearOrder, B: LinearOrder](f: A -> B, a: A, b: A, x: A) {
    is_strict_monotone(f) and open_interval(a, b, x) implies open_interval(f(a), f(b), f(x))
} by {
    if is_strict_monotone(f) and open_interval(a, b, x) {
        a < x
        strict_monotone_step(f, a, x)
        f(a) < f(x)
        x < b
        strict_monotone_step(f, x, b)
        f(x) < f(b)
        open_interval(f(a), f(b), f(x))
    }
}

/// A monotone and strict monotone map sends left-open intervals into left-open intervals.
theorem monotone_left_open_interval_image[A: LinearOrder, B: LinearOrder](f: A -> B, a: A, b: A, x: A) {
    is_monotone(f) and is_strict_monotone(f) and left_open_interval(a, b, x)
    implies left_open_interval(f(a), f(b), f(x))
} by {
    if is_monotone(f) and is_strict_monotone(f) and left_open_interval(a, b, x) {
        a < x
        strict_monotone_step(f, a, x)
        f(a) < f(x)
        x <= b
        monotone_step(f, x, b)
        f(x) <= f(b)
        left_open_interval(f(a), f(b), f(x))
    }
}

/// A monotone and strict monotone map sends right-open intervals into right-open intervals.
theorem monotone_right_open_interval_image[A: LinearOrder, B: LinearOrder](f: A -> B, a: A, b: A, x: A) {
    is_monotone(f) and is_strict_monotone(f) and right_open_interval(a, b, x)
    implies right_open_interval(f(a), f(b), f(x))
} by {
    if is_monotone(f) and is_strict_monotone(f) and right_open_interval(a, b, x) {
        a <= x
        monotone_step(f, a, x)
        f(a) <= f(x)
        x < b
        strict_monotone_step(f, x, b)
        f(x) < f(b)
        right_open_interval(f(a), f(b), f(x))
    }
}

/// An antitone map sends closed intervals into closed intervals between the reversed image endpoints.
theorem antitone_closed_interval_image[A: LinearOrder, B: LinearOrder](f: A -> B, a: A, b: A, x: A) {
    is_antitone(f) and closed_interval(a, b, x) implies closed_interval(f(b), f(a), f(x))
} by {
    if is_antitone(f) and closed_interval(a, b, x) {
        a <= x
        antitone_step(f, a, x)
        f(x) <= f(a)
        x <= b
        antitone_step(f, x, b)
        f(b) <= f(x)
        closed_interval(f(b), f(a), f(x))
    }
}

/// A strict antitone map sends open intervals into open intervals between the reversed image endpoints.
theorem strict_antitone_open_interval_image[A: LinearOrder, B: LinearOrder](f: A -> B, a: A, b: A, x: A) {
    is_strict_antitone(f) and open_interval(a, b, x) implies open_interval(f(b), f(a), f(x))
} by {
    if is_strict_antitone(f) and open_interval(a, b, x) {
        a < x
        strict_antitone_step(f, a, x)
        f(x) < f(a)
        x < b
        strict_antitone_step(f, x, b)
        f(b) < f(x)
        open_interval(f(b), f(a), f(x))
    }
}

/// An antitone and strict antitone map sends left-open intervals into right-open intervals.
theorem antitone_left_open_interval_image[A: LinearOrder, B: LinearOrder](f: A -> B, a: A, b: A, x: A) {
    is_antitone(f) and is_strict_antitone(f) and left_open_interval(a, b, x)
    implies right_open_interval(f(b), f(a), f(x))
} by {
    if is_antitone(f) and is_strict_antitone(f) and left_open_interval(a, b, x) {
        a < x
        strict_antitone_step(f, a, x)
        f(x) < f(a)
        x <= b
        antitone_step(f, x, b)
        f(b) <= f(x)
        right_open_interval(f(b), f(a), f(x))
    }
}

/// An antitone and strict antitone map sends right-open intervals into left-open intervals.
theorem antitone_right_open_interval_image[A: LinearOrder, B: LinearOrder](f: A -> B, a: A, b: A, x: A) {
    is_antitone(f) and is_strict_antitone(f) and right_open_interval(a, b, x)
    implies left_open_interval(f(b), f(a), f(x))
} by {
    if is_antitone(f) and is_strict_antitone(f) and right_open_interval(a, b, x) {
        a <= x
        antitone_step(f, a, x)
        f(x) <= f(a)
        x < b
        strict_antitone_step(f, x, b)
        f(b) < f(x)
        left_open_interval(f(b), f(a), f(x))
    }
}

/// An order embedding sends closed intervals into closed intervals between image endpoints.
theorem order_embedding_closed_interval_image[A: LinearOrder, B: LinearOrder](f: A -> B, a: A, b: A, x: A) {
    is_order_embedding(f) and closed_interval(a, b, x) implies closed_interval(f(a), f(b), f(x))
} by {
    if is_order_embedding(f) and closed_interval(a, b, x) {
        order_embedding_is_monotone(f)
        is_monotone(f)
        monotone_closed_interval_image(f, a, b, x)
        closed_interval(f(a), f(b), f(x))
    }
}

/// An order embedding sends open intervals into open intervals between image endpoints.
theorem order_embedding_open_interval_image[A: LinearOrder, B: LinearOrder](f: A -> B, a: A, b: A, x: A) {
    is_order_embedding(f) and open_interval(a, b, x) implies open_interval(f(a), f(b), f(x))
} by {
    if is_order_embedding(f) and open_interval(a, b, x) {
        order_embedding_is_strict_monotone(f)
        is_strict_monotone(f)
        strict_monotone_open_interval_image(f, a, b, x)
        open_interval(f(a), f(b), f(x))
    }
}

/// An order embedding sends left-open intervals into left-open intervals between image endpoints.
theorem order_embedding_left_open_interval_image[A: LinearOrder, B: LinearOrder](f: A -> B, a: A, b: A, x: A) {
    is_order_embedding(f) and left_open_interval(a, b, x) implies left_open_interval(f(a), f(b), f(x))
} by {
    if is_order_embedding(f) and left_open_interval(a, b, x) {
        order_embedding_is_monotone(f)
        is_monotone(f)
        order_embedding_is_strict_monotone(f)
        is_strict_monotone(f)
        monotone_left_open_interval_image(f, a, b, x)
        left_open_interval(f(a), f(b), f(x))
    }
}

/// An order embedding sends right-open intervals into right-open intervals between image endpoints.
theorem order_embedding_right_open_interval_image[A: LinearOrder, B: LinearOrder](f: A -> B, a: A, b: A, x: A) {
    is_order_embedding(f) and right_open_interval(a, b, x) implies right_open_interval(f(a), f(b), f(x))
} by {
    if is_order_embedding(f) and right_open_interval(a, b, x) {
        order_embedding_is_monotone(f)
        is_monotone(f)
        order_embedding_is_strict_monotone(f)
        is_strict_monotone(f)
        monotone_right_open_interval_image(f, a, b, x)
        right_open_interval(f(a), f(b), f(x))
    }
}

/// An order embedding reflects membership in a closed interval between image endpoints.
theorem order_embedding_closed_interval_preimage[A: LinearOrder, B: LinearOrder](f: A -> B, a: A, b: A, x: A) {
    is_order_embedding(f) and closed_interval(f(a), f(b), f(x)) implies closed_interval(a, b, x)
} by {
    if is_order_embedding(f) and closed_interval(f(a), f(b), f(x)) {
        f(a) <= f(x)
        order_embedding_reflects_lte(f, a, x)
        a <= x
        f(x) <= f(b)
        order_embedding_reflects_lte(f, x, b)
        x <= b
        closed_interval(a, b, x)
    }
}

/// An order embedding reflects membership in an open interval between image endpoints.
theorem order_embedding_open_interval_preimage[A: LinearOrder, B: LinearOrder](f: A -> B, a: A, b: A, x: A) {
    is_order_embedding(f) and open_interval(f(a), f(b), f(x)) implies open_interval(a, b, x)
} by {
    if is_order_embedding(f) and open_interval(f(a), f(b), f(x)) {
        f(a) < f(x)
        f(a) <= f(x)
        order_embedding_reflects_lte(f, a, x)
        a <= x
        f(a) != f(x)
        if a = x {
            f(a) = f(x)
            false
        }
        a < x
        f(x) < f(b)
        f(x) <= f(b)
        order_embedding_reflects_lte(f, x, b)
        x <= b
        f(x) != f(b)
        if x = b {
            f(x) = f(b)
            false
        }
        x < b
        open_interval(a, b, x)
    }
}

/// An order embedding reflects membership in a left-open interval between image endpoints.
theorem order_embedding_left_open_interval_preimage[A: LinearOrder, B: LinearOrder](f: A -> B, a: A, b: A, x: A) {
    is_order_embedding(f) and left_open_interval(f(a), f(b), f(x)) implies left_open_interval(a, b, x)
} by {
    if is_order_embedding(f) and left_open_interval(f(a), f(b), f(x)) {
        f(a) < f(x)
        f(a) <= f(x)
        order_embedding_reflects_lte(f, a, x)
        a <= x
        f(a) != f(x)
        if a = x {
            f(a) = f(x)
            false
        }
        a < x
        f(x) <= f(b)
        order_embedding_reflects_lte(f, x, b)
        x <= b
        left_open_interval(a, b, x)
    }
}

/// An order embedding reflects membership in a right-open interval between image endpoints.
theorem order_embedding_right_open_interval_preimage[A: LinearOrder, B: LinearOrder](f: A -> B, a: A, b: A, x: A) {
    is_order_embedding(f) and right_open_interval(f(a), f(b), f(x)) implies right_open_interval(a, b, x)
} by {
    if is_order_embedding(f) and right_open_interval(f(a), f(b), f(x)) {
        f(a) <= f(x)
        order_embedding_reflects_lte(f, a, x)
        a <= x
        f(x) < f(b)
        f(x) <= f(b)
        order_embedding_reflects_lte(f, x, b)
        x <= b
        f(x) != f(b)
        if x = b {
            f(x) = f(b)
            false
        }
        x < b
        right_open_interval(a, b, x)
    }
}

/// An order embedding identifies closed-interval membership with image closed-interval membership.
theorem order_embedding_closed_interval_iff[A: LinearOrder, B: LinearOrder](f: A -> B, a: A, b: A, x: A) {
    is_order_embedding(f) implies closed_interval(f(a), f(b), f(x)) = closed_interval(a, b, x)
} by {
    if is_order_embedding(f) {
        if closed_interval(f(a), f(b), f(x)) {
            order_embedding_closed_interval_preimage(f, a, b, x)
            closed_interval(a, b, x)
        }
        if closed_interval(a, b, x) {
            order_embedding_closed_interval_image(f, a, b, x)
            closed_interval(f(a), f(b), f(x))
        }
        closed_interval(f(a), f(b), f(x)) = closed_interval(a, b, x)
    }
}

/// An order embedding identifies open-interval membership with image open-interval membership.
theorem order_embedding_open_interval_iff[A: LinearOrder, B: LinearOrder](f: A -> B, a: A, b: A, x: A) {
    is_order_embedding(f) implies open_interval(f(a), f(b), f(x)) = open_interval(a, b, x)
} by {
    if is_order_embedding(f) {
        if open_interval(f(a), f(b), f(x)) {
            order_embedding_open_interval_preimage(f, a, b, x)
            open_interval(a, b, x)
        }
        if open_interval(a, b, x) {
            order_embedding_open_interval_image(f, a, b, x)
            open_interval(f(a), f(b), f(x))
        }
        open_interval(f(a), f(b), f(x)) = open_interval(a, b, x)
    }
}

/// An order embedding identifies left-open interval membership with image left-open interval membership.
theorem order_embedding_left_open_interval_iff[A: LinearOrder, B: LinearOrder](f: A -> B, a: A, b: A, x: A) {
    is_order_embedding(f) implies left_open_interval(f(a), f(b), f(x)) = left_open_interval(a, b, x)
} by {
    if is_order_embedding(f) {
        if left_open_interval(f(a), f(b), f(x)) {
            order_embedding_left_open_interval_preimage(f, a, b, x)
            left_open_interval(a, b, x)
        }
        if left_open_interval(a, b, x) {
            order_embedding_left_open_interval_image(f, a, b, x)
            left_open_interval(f(a), f(b), f(x))
        }
        left_open_interval(f(a), f(b), f(x)) = left_open_interval(a, b, x)
    }
}

/// An order embedding identifies right-open interval membership with image right-open interval membership.
theorem order_embedding_right_open_interval_iff[A: LinearOrder, B: LinearOrder](f: A -> B, a: A, b: A, x: A) {
    is_order_embedding(f) implies right_open_interval(f(a), f(b), f(x)) = right_open_interval(a, b, x)
} by {
    if is_order_embedding(f) {
        if right_open_interval(f(a), f(b), f(x)) {
            order_embedding_right_open_interval_preimage(f, a, b, x)
            right_open_interval(a, b, x)
        }
        if right_open_interval(a, b, x) {
            order_embedding_right_open_interval_image(f, a, b, x)
            right_open_interval(f(a), f(b), f(x))
        }
        right_open_interval(f(a), f(b), f(x)) = right_open_interval(a, b, x)
    }
}

/// An order embedding preserves clamping to closed intervals.
theorem order_embedding_clamp_image[A: LinearOrder, B: LinearOrder](f: A -> B, a: A, b: A, x: A) {
    is_order_embedding(f) implies f(clamp(a, b, x)) = clamp(f(a), f(b), f(x))
} by {
    if is_order_embedding(f) {
        order_embedding_max_image(f, a, x)
        f(a.max(x)) = f(a).max(f(x))
        order_embedding_min_image(f, a.max(x), b)
        f(a.max(x).min(b)) = f(a.max(x)).min(f(b))
        f(a.max(x).min(b)) = f(a).max(f(x)).min(f(b))
        clamp(a, b, x) = a.max(x).min(b)
        clamp(f(a), f(b), f(x)) = f(a).max(f(x)).min(f(b))
        f(clamp(a, b, x)) = clamp(f(a), f(b), f(x))
    }
}
