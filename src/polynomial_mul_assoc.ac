from nat import Nat, add_sub, add_imp_sub, add_imp_sub_left, lt_and_lte
from semiring import Semiring
from comm_ring import CommRing
from data.basic.functions import function_extensionality
from polynomial import Polynomial, polynomial_mul, polynomial_mul_coeff,
    polynomial_mul_term_coeff, polynomial_mul_coeff_apply, polynomial_ext_pointwise
from data.nat.nat_range_sum import range_sum, range_sum_zero, range_sum_suc, range_sum_congr
from data.nat.nat_range_sum_interchange import scale_fn, range_sum_scale
from data.nat.nat_triangular_sum import diag_fn, diag_sum, tri_diag_sum, row_upto_sum, tri_row_sum,
    tri_sum_eq
from polynomial_mul_comm import mul_coeff_eq_range_sum

/// A summand scaled on the right by a fixed factor.
define scale_right_fn[R: Semiring](f: Nat -> R, c: R, i: Nat) -> R {
    f(i) * c
}

/// A constant factor comes out of a range sum on the right.
///
/// The mirror of `range_sum_scale`. A convolution rearrangement needs both sides, since one
/// association puts the spare factor on the left and the other puts it on the right.
theorem range_sum_scale_right[R: Semiring](f: Nat -> R, c: R, n: Nat) {
    range_sum(scale_right_fn(f, c), n) = range_sum(f, n) * c
} by {
    define p(k: Nat) -> Bool {
        range_sum(scale_right_fn(f, c), k) = range_sum(f, k) * c
    }
    range_sum_zero(scale_right_fn(f, c))
    range_sum_zero(f)
    (R.0 * c = R.0)
    p(Nat.0)
    forall(k: Nat) {
        if p(k) {
            (range_sum(scale_right_fn(f, c), k) = range_sum(f, k) * c)
            range_sum_suc(scale_right_fn(f, c), k)
            (range_sum(scale_right_fn(f, c), k.suc)
                = range_sum(scale_right_fn(f, c), k) + scale_right_fn(f, c)(k))
            (scale_right_fn(f, c, k) = f(k) * c)
            range_sum_suc(f, k)
            (range_sum(f, k.suc) = range_sum(f, k) + f(k))
            ((range_sum(f, k) + f(k)) * c = range_sum(f, k) * c + f(k) * c)
            range_sum(scale_right_fn(f, c), k.suc) = range_sum(f, k.suc) * c
            p(k.suc)
        }
        (p(k) implies p(k.suc))
    }
    p(Nat.0) and forall(k: Nat) {
        p(k) implies p(k.suc)
    }
    Nat.induction(p)
    p(n)
}

/// Splitting a total three ways: what is left after two parts is the third.
///
/// The index identity the anti-diagonal form needs. Written through additions rather than
/// directly, since neither cancellation is found on its own.
theorem sub_sub_diag(k: Nat, i: Nat, j: Nat) {
    j <= i and i <= k implies k - j - (i - j) = k - i
} by {
    if j <= i and i <= k {
        add_sub(i, j)
        ((i - j) + j = i)
        add_sub(k, i)
        ((k - i) + i = k)
        ((k - i) + ((i - j) + j) = k)
        (((k - i) + (i - j)) + j = k)
        add_imp_sub_left(j, (k - i) + (i - j), k)
        (k - j = (k - i) + (i - j))
        add_imp_sub((k - i), (i - j), k - j)
        ((k - j) - (i - j) = k - i)
    }
}

/// The summand of a triple product, indexed by the first two exponents.
///
/// The third exponent is whatever is left of the total, so the triangle of pairs with total at
/// most `k` runs over exactly the triples summing to `k`.
define assoc_term[R: CommRing](
    p: Polynomial[R], q: Polynomial[R], r: Polynomial[R], k: Nat, a: Nat, b: Nat
) -> R {
    p.coeff(a) * (q.coeff(b) * r.coeff(k - a - b))
}

/// Each row of the triangle is the first factor times a convolution coefficient.
theorem assoc_term_row[R: CommRing](
    p: Polynomial[R], q: Polynomial[R], r: Polynomial[R], k: Nat, a: Nat
) {
    row_upto_sum(assoc_term(p, q, r, k), k, a)
        = p.coeff(a) * polynomial_mul_coeff(q, r, k - a)
} by {
    forall(b: Nat) {
        (assoc_term(p, q, r, k, a, b)
            = p.coeff(a) * (q.coeff(b) * r.coeff(k - a - b)))
        (polynomial_mul_term_coeff(q, r, k - a, b) = q.coeff(b) * r.coeff(k - a - b))
        (scale_fn(p.coeff(a), polynomial_mul_term_coeff(q, r, k - a))(b)
            = p.coeff(a) * polynomial_mul_term_coeff(q, r, k - a)(b))
        (assoc_term(p, q, r, k)(a)(b)
            = scale_fn(p.coeff(a), polynomial_mul_term_coeff(q, r, k - a))(b))
    }
    function_extensionality[Nat, R](assoc_term(p, q, r, k)(a),
        scale_fn(p.coeff(a), polynomial_mul_term_coeff(q, r, k - a)))
    (assoc_term(p, q, r, k)(a)
        = scale_fn(p.coeff(a), polynomial_mul_term_coeff(q, r, k - a)))
    (row_upto_sum(assoc_term(p, q, r, k), k, a)
        = range_sum(assoc_term(p, q, r, k)(a), (k - a).suc))
    range_sum_scale(p.coeff(a), polynomial_mul_term_coeff(q, r, k - a), (k - a).suc)
    (range_sum(scale_fn(p.coeff(a), polynomial_mul_term_coeff(q, r, k - a)), (k - a).suc)
        = p.coeff(a) * range_sum(polynomial_mul_term_coeff(q, r, k - a), (k - a).suc))
    mul_coeff_eq_range_sum(q, r, k - a)
    (polynomial_mul_coeff(q, r, k - a)
        = range_sum(polynomial_mul_term_coeff(q, r, k - a), (k - a).suc))
    (row_upto_sum(assoc_term(p, q, r, k), k, a)
        = p.coeff(a) * polynomial_mul_coeff(q, r, k - a))
}

/// The triangle summed by rows is the right-associated product coefficient.
theorem tri_row_sum_assoc[R: CommRing](
    p: Polynomial[R], q: Polynomial[R], r: Polynomial[R], k: Nat
) {
    (tri_row_sum(assoc_term(p, q, r, k), k)
        = polynomial_mul_coeff(p, polynomial_mul(q, r), k))
} by {
    forall(a: Nat) {
        if a < k.suc {
            assoc_term_row(p, q, r, k, a)
            (row_upto_sum(assoc_term(p, q, r, k), k, a)
                = p.coeff(a) * polynomial_mul_coeff(q, r, k - a))
            polynomial_mul_coeff_apply(q, r, k - a)
            (polynomial_mul(q, r).coeff(k - a) = polynomial_mul_coeff(q, r, k - a))
            (polynomial_mul_term_coeff(p, polynomial_mul(q, r), k, a)
                = p.coeff(a) * polynomial_mul(q, r).coeff(k - a))
            (row_upto_sum(assoc_term(p, q, r, k), k)(a)
                = polynomial_mul_term_coeff(p, polynomial_mul(q, r), k)(a))
        }
        (a < k.suc implies row_upto_sum(assoc_term(p, q, r, k), k)(a)
            = polynomial_mul_term_coeff(p, polynomial_mul(q, r), k)(a))
    }
    range_sum_congr(row_upto_sum(assoc_term(p, q, r, k), k),
        polynomial_mul_term_coeff(p, polynomial_mul(q, r), k), k.suc)
    (tri_row_sum(assoc_term(p, q, r, k), k)
        = range_sum(row_upto_sum(assoc_term(p, q, r, k), k), k.suc))
    (range_sum(row_upto_sum(assoc_term(p, q, r, k), k), k.suc)
        = range_sum(polynomial_mul_term_coeff(p, polynomial_mul(q, r), k), k.suc))
    mul_coeff_eq_range_sum(p, polynomial_mul(q, r), k)
    (polynomial_mul_coeff(p, polynomial_mul(q, r), k)
        = range_sum(polynomial_mul_term_coeff(p, polynomial_mul(q, r), k), k.suc))
    (tri_row_sum(assoc_term(p, q, r, k), k)
        = polynomial_mul_coeff(p, polynomial_mul(q, r), k))
}

/// Each anti-diagonal of the triangle is a convolution coefficient times the third factor.
///
/// On the anti-diagonal of total `i` the third exponent is `k - i` throughout, since the first
/// two already add to `i`. That is what lets the third factor come out of the sum.
theorem assoc_term_diag[R: CommRing](
    p: Polynomial[R], q: Polynomial[R], r: Polynomial[R], k: Nat, i: Nat
) {
    i <= k implies diag_sum(assoc_term(p, q, r, k), i)
        = polynomial_mul_coeff(p, q, i) * r.coeff(k - i)
} by {
    if i <= k {
        forall(j: Nat) {
            if j < i.suc {
                j <= i
                (diag_fn(assoc_term(p, q, r, k), i, j)
                    = assoc_term(p, q, r, k)(j)(i - j))
                (assoc_term(p, q, r, k, j, i - j)
                    = p.coeff(j) * (q.coeff(i - j) * r.coeff(k - j - (i - j))))
                sub_sub_diag(k, i, j)
                (k - j - (i - j) = k - i)
                (p.coeff(j) * (q.coeff(i - j) * r.coeff(k - i))
                    = (p.coeff(j) * q.coeff(i - j)) * r.coeff(k - i))
                (polynomial_mul_term_coeff(p, q, i, j) = p.coeff(j) * q.coeff(i - j))
                (scale_right_fn(polynomial_mul_term_coeff(p, q, i), r.coeff(k - i))(j)
                    = polynomial_mul_term_coeff(p, q, i)(j) * r.coeff(k - i))
                (diag_fn(assoc_term(p, q, r, k), i)(j)
                    = scale_right_fn(polynomial_mul_term_coeff(p, q, i),
                        r.coeff(k - i))(j))
            }
            (j < i.suc implies diag_fn(assoc_term(p, q, r, k), i)(j)
                = scale_right_fn(polynomial_mul_term_coeff(p, q, i), r.coeff(k - i))(j))
        }
        range_sum_congr(diag_fn(assoc_term(p, q, r, k), i),
            scale_right_fn(polynomial_mul_term_coeff(p, q, i), r.coeff(k - i)), i.suc)
        (diag_sum(assoc_term(p, q, r, k), i)
            = range_sum(diag_fn(assoc_term(p, q, r, k), i), i.suc))
        (range_sum(diag_fn(assoc_term(p, q, r, k), i), i.suc)
            = range_sum(scale_right_fn(polynomial_mul_term_coeff(p, q, i),
                r.coeff(k - i)), i.suc))
        range_sum_scale_right(polynomial_mul_term_coeff(p, q, i), r.coeff(k - i), i.suc)
        (range_sum(scale_right_fn(polynomial_mul_term_coeff(p, q, i), r.coeff(k - i)), i.suc)
            = range_sum(polynomial_mul_term_coeff(p, q, i), i.suc) * r.coeff(k - i))
        mul_coeff_eq_range_sum(p, q, i)
        (polynomial_mul_coeff(p, q, i)
            = range_sum(polynomial_mul_term_coeff(p, q, i), i.suc))
        (diag_sum(assoc_term(p, q, r, k), i)
            = polynomial_mul_coeff(p, q, i) * r.coeff(k - i))
    }
}

/// The triangle summed by anti-diagonals is the left-associated product coefficient.
theorem tri_diag_sum_assoc[R: CommRing](
    p: Polynomial[R], q: Polynomial[R], r: Polynomial[R], k: Nat
) {
    tri_diag_sum(assoc_term(p, q, r, k), k)
        = polynomial_mul_coeff(polynomial_mul(p, q), r, k)
} by {
    forall(i: Nat) {
        if i < k.suc {
            i <= k
            assoc_term_diag(p, q, r, k, i)
            (diag_sum(assoc_term(p, q, r, k), i)
                = polynomial_mul_coeff(p, q, i) * r.coeff(k - i))
            polynomial_mul_coeff_apply(p, q, i)
            (polynomial_mul(p, q).coeff(i) = polynomial_mul_coeff(p, q, i))
            (polynomial_mul_term_coeff(polynomial_mul(p, q), r, k, i)
                = polynomial_mul(p, q).coeff(i) * r.coeff(k - i))
            (diag_sum(assoc_term(p, q, r, k))(i)
                = polynomial_mul_term_coeff(polynomial_mul(p, q), r, k)(i))
        }
        (i < k.suc implies diag_sum(assoc_term(p, q, r, k))(i)
            = polynomial_mul_term_coeff(polynomial_mul(p, q), r, k)(i))
    }
    range_sum_congr(diag_sum(assoc_term(p, q, r, k)),
        polynomial_mul_term_coeff(polynomial_mul(p, q), r, k), k.suc)
    (tri_diag_sum(assoc_term(p, q, r, k), k)
        = range_sum(diag_sum(assoc_term(p, q, r, k)), k.suc))
    (range_sum(diag_sum(assoc_term(p, q, r, k)), k.suc)
        = range_sum(polynomial_mul_term_coeff(polynomial_mul(p, q), r, k), k.suc))
    mul_coeff_eq_range_sum(polynomial_mul(p, q), r, k)
    (polynomial_mul_coeff(polynomial_mul(p, q), r, k)
        = range_sum(polynomial_mul_term_coeff(polynomial_mul(p, q), r, k), k.suc))
    (tri_diag_sum(assoc_term(p, q, r, k), k)
        = polynomial_mul_coeff(polynomial_mul(p, q), r, k))
}

/// Polynomial multiplication is associative.
///
/// Both associations are the same sum over triples of exponents adding to the degree: the
/// left-associated one groups them by the total of the first two, the right-associated one by the
/// first alone. Those are the anti-diagonals and the rows of the same triangle.
theorem polynomial_mul_assoc[R: CommRing](
    p: Polynomial[R], q: Polynomial[R], r: Polynomial[R]
) {
    polynomial_mul(polynomial_mul(p, q), r) = polynomial_mul(p, polynomial_mul(q, r))
} by {
    forall(k: Nat) {
        tri_sum_eq(assoc_term(p, q, r, k), k)
        (tri_diag_sum(assoc_term(p, q, r, k), k) = tri_row_sum(assoc_term(p, q, r, k), k))
        tri_diag_sum_assoc(p, q, r, k)
        (tri_diag_sum(assoc_term(p, q, r, k), k)
            = polynomial_mul_coeff(polynomial_mul(p, q), r, k))
        tri_row_sum_assoc(p, q, r, k)
        (tri_row_sum(assoc_term(p, q, r, k), k)
            = polynomial_mul_coeff(p, polynomial_mul(q, r), k))
        (polynomial_mul_coeff(polynomial_mul(p, q), r, k)
            = polynomial_mul_coeff(p, polynomial_mul(q, r), k))
        polynomial_mul_coeff_apply(polynomial_mul(p, q), r, k)
        (polynomial_mul(polynomial_mul(p, q), r).coeff(k)
            = polynomial_mul_coeff(polynomial_mul(p, q), r, k))
        polynomial_mul_coeff_apply(p, polynomial_mul(q, r), k)
        (polynomial_mul(p, polynomial_mul(q, r)).coeff(k)
            = polynomial_mul_coeff(p, polynomial_mul(q, r), k))
        (polynomial_mul(polynomial_mul(p, q), r).coeff(k)
            = polynomial_mul(p, polynomial_mul(q, r)).coeff(k))
    }
    polynomial_ext_pointwise(polynomial_mul(polynomial_mul(p, q), r),
        polynomial_mul(p, polynomial_mul(q, r)))
    polynomial_mul(polynomial_mul(p, q), r) = polynomial_mul(p, polynomial_mul(q, r))
}
